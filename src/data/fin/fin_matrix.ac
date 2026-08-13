from nat import Nat
from data.fin.fin import Fin, fin_skip, fin_zero
from algebra.add import Add
from algebra.zero import Zero
from algebra.neg import Neg
from algebra.add_semigroup import AddSemigroup
from algebra.add_comm_semigroup import AddCommSemigroup
from algebra.add_monoid import AddMonoid
from algebra.add_comm_monoid import AddCommMonoid
from algebra.add_group import AddGroup
from algebra.add_comm_group import AddCommGroup
from algebra.ring.ring import Ring, alternating_sign
from comm_ring import CommRing
from semiring import Semiring
from algebra.module.module import Module, module_smul_add_left, module_smul_add_right, module_smul_assoc,
    module_smul_one, module_smul_zero_left, module_smul_zero_right, module_smul_neg_left, module_smul_neg_right
from data.basic.functions import function_extensionality
from data.fin.fin_vector import fin_vector_add, fin_vector_neg, fin_vector_smul,
    fin_vector_sub, fin_vector_zero
from list import List, map, sum
from data.fin.fin_sum import fin_sum, fin_pointwise_add, fin_zero_function, fin_sum_add,
    fin_sum_pointwise_eq, fin_sum_zero_function, fin_sum_zero_of_bound_zero
from data.fin.fin_sum_enum import fin_sum_single_nonzero

/// An `m` by `n` matrix with entries in `A`, indexed by finite row and column sets.
structure Matrix[A, m: Nat, n: Nat] {
    /// The entry function of the matrix.
    entry: (Fin[m], Fin[n]) -> A
}

/// The entry of a matrix.
define matrix_entry[A](m: Nat, n: Nat, a: Matrix[A, m, n], i: Fin[m], j: Fin[n]) -> A {
    a.entry(i, j)
}

/// The entry function of the zero matrix.
define matrix_zero_entry[A: Zero](m: Nat, n: Nat, i: Fin[m], j: Fin[n]) -> A {
    A.0
}

/// The zero `m` by `n` matrix.
define matrix_zero[A: Zero](m: Nat, n: Nat) -> Matrix[A, m, n] {
    Matrix[A, m, n].new(matrix_zero_entry[A](m, n))
}

/// The entry function of the pointwise sum of two matrices.
define matrix_add_entry[A: Add](m: Nat, n: Nat, a: Matrix[A, m, n], b: Matrix[A, m, n], i: Fin[m], j: Fin[n]) -> A {
    a.entry(i, j) + b.entry(i, j)
}

/// Pointwise addition of `m` by `n` matrices.
define matrix_add[A: Add](m: Nat, n: Nat, a: Matrix[A, m, n], b: Matrix[A, m, n]) -> Matrix[A, m, n] {
    Matrix[A, m, n].new(matrix_add_entry[A](m, n, a, b))
}

/// The entry function of the pointwise negation of a matrix.
define matrix_neg_entry[A: Neg](m: Nat, n: Nat, a: Matrix[A, m, n], i: Fin[m], j: Fin[n]) -> A {
    -a.entry(i, j)
}

/// Pointwise negation of an `m` by `n` matrix.
define matrix_neg[A: Neg](m: Nat, n: Nat, a: Matrix[A, m, n]) -> Matrix[A, m, n] {
    Matrix[A, m, n].new(matrix_neg_entry[A](m, n, a))
}

/// The entry function of the pointwise difference of two matrices.
define matrix_sub_entry[A: AddGroup](m: Nat, n: Nat, a: Matrix[A, m, n], b: Matrix[A, m, n], i: Fin[m], j: Fin[n]) -> A {
    a.entry(i, j) - b.entry(i, j)
}

/// Pointwise subtraction of `m` by `n` matrices.
define matrix_sub[A: AddGroup](m: Nat, n: Nat, a: Matrix[A, m, n], b: Matrix[A, m, n]) -> Matrix[A, m, n] {
    Matrix[A, m, n].new(matrix_sub_entry[A](m, n, a, b))
}

/// The entry function of the scalar multiple of a matrix.
define matrix_smul_entry[R: Ring, M: AddCommGroup](
    carrier: Module[R, M], m: Nat, n: Nat, r: R, a: Matrix[M, m, n], i: Fin[m], j: Fin[n]
) -> M {
    carrier.smul(r, a.entry(i, j))
}

/// Scalar multiplication of a matrix with entries in a module.
define matrix_smul[R: Ring, M: AddCommGroup](
    carrier: Module[R, M], m: Nat, n: Nat, r: R, a: Matrix[M, m, n]
) -> Matrix[M, m, n] {
    Matrix[M, m, n].new(matrix_smul_entry[R, M](carrier, m, n, r, a))
}

/// The entry function of the transpose of a matrix.
define matrix_transpose_entry[A](m: Nat, n: Nat, a: Matrix[A, m, n], j: Fin[n], i: Fin[m]) -> A {
    a.entry(i, j)
}

/// The transpose of an `m` by `n` matrix.
define matrix_transpose[A](m: Nat, n: Nat, a: Matrix[A, m, n]) -> Matrix[A, n, m] {
    Matrix[A, n, m].new(matrix_transpose_entry[A](m, n, a))
}

/// The `i`th row of an `m` by `n` matrix, as a coordinate vector of length `n`.
define matrix_row[A](m: Nat, n: Nat, a: Matrix[A, m, n], i: Fin[m], j: Fin[n]) -> A {
    a.entry(i, j)
}

/// The `j`th column of an `m` by `n` matrix, as a coordinate vector of length `m`.
define matrix_col[A](m: Nat, n: Nat, a: Matrix[A, m, n], j: Fin[n], i: Fin[m]) -> A {
    a.entry(i, j)
}

/// The `k`th summand in the product of an `m` by `n` matrix and an `n` by `p` matrix.
define matrix_mul_term[S: Semiring](
    m: Nat,
    n: Nat,
    p: Nat,
    a: Matrix[S, m, n],
    b: Matrix[S, n, p],
    i: Fin[m],
    j: Fin[p],
    k: Fin[n]
) -> S {
    matrix_entry[S](m, n, a, i, k) * matrix_entry[S](n, p, b, k, j)
}

/// The entry function of the product of an `m` by `n` matrix and an `n` by `p` matrix.
define matrix_mul_entry[S: Semiring](
    m: Nat,
    n: Nat,
    p: Nat,
    a: Matrix[S, m, n],
    b: Matrix[S, n, p],
    i: Fin[m],
    j: Fin[p]
) -> S {
    fin_sum[S](n, matrix_mul_term[S](m, n, p, a, b, i, j))
}

/// The product of an `m` by `n` matrix and an `n` by `p` matrix.
define matrix_mul[S: Semiring](m: Nat, n: Nat, p: Nat, a: Matrix[S, m, n], b: Matrix[S, n, p]) -> Matrix[S, m, p] {
    Matrix[S, m, p].new(matrix_mul_entry[S](m, n, p, a, b))
}

/// The entry function of the identity square matrix.
define matrix_one_entry[S: Semiring](n: Nat, i: Fin[n], j: Fin[n]) -> S {
    if i = j {
        S.1
    } else {
        S.0
    }
}

/// The identity `n` by `n` matrix.
define matrix_one[S: Semiring](n: Nat) -> Matrix[S, n, n] {
    Matrix[S, n, n].new(matrix_one_entry[S](n))
}

/// Multiplication of two square matrices of the same size.
define square_matrix_mul[S: Semiring](n: Nat, a: Matrix[S, n, n], b: Matrix[S, n, n]) -> Matrix[S, n, n] {
    matrix_mul[S](n, n, n, a, b)
}

/// A natural-number power of a square matrix.
define matrix_pow[S: Semiring](n: Nat, a: Matrix[S, n, n], exp: Nat) -> Matrix[S, n, n] {
    match exp {
        Nat.zero {
            matrix_one[S](n)
        }
        Nat.suc(k) {
            square_matrix_mul[S](n, a, matrix_pow[S](n, a, k))
        }
    }
}

/// The entry function of the minor obtained by deleting one row and one column.
define matrix_minor_entry[A](n: Nat, a: Matrix[A, n.suc, n.suc], row: Fin[n.suc], col: Fin[n.suc], i: Fin[n], j: Fin[n]) -> A {
    matrix_entry[A](n.suc, n.suc, a, fin_skip(n, row, i), fin_skip(n, col, j))
}

/// The minor obtained by deleting one row and one column.
define matrix_minor[A](n: Nat, a: Matrix[A, n.suc, n.suc], row: Fin[n.suc], col: Fin[n.suc]) -> Matrix[A, n, n] {
    Matrix[A, n, n].new(matrix_minor_entry[A](n, a, row, col))
}

/// The sign attached to a matrix cofactor.
define matrix_cofactor_sign[R: Ring](n: Nat, row: Fin[n], col: Fin[n]) -> R {
    alternating_sign[R](row.value + col.value)
}

/// The cofactor formed from an externally supplied determinant on minors.
define matrix_cofactor_with[R: Ring](n: Nat, det_small: Matrix[R, n, n] -> R, a: Matrix[R, n.suc, n.suc], row: Fin[n.suc], col: Fin[n.suc]) -> R {
    matrix_cofactor_sign[R](n.suc, row, col) * det_small(matrix_minor[R](n, a, row, col))
}

/// One summand in a Laplace expansion along a fixed row.
define matrix_laplace_row_term[R: Ring](n: Nat, det_small: Matrix[R, n, n] -> R, a: Matrix[R, n.suc, n.suc], row: Fin[n.suc], col: Fin[n.suc]) -> R {
    matrix_entry[R](n.suc, n.suc, a, row, col) * matrix_cofactor_with[R](n, det_small, a, row, col)
}

/// The Laplace expansion along a fixed row, relative to a determinant on minors.
define matrix_laplace_row[R: Ring](n: Nat, det_small: Matrix[R, n, n] -> R, a: Matrix[R, n.suc, n.suc], row: Fin[n.suc]) -> R {
    fin_sum[R](n.suc, matrix_laplace_row_term[R](n, det_small, a, row))
}

/// One summand in the first-row Laplace expansion.
define matrix_laplace_first_row_term[R: Ring](n: Nat, det_small: Matrix[R, n, n] -> R, a: Matrix[R, n.suc, n.suc], col: Fin[n.suc]) -> R {
    matrix_laplace_row_term[R](n, det_small, a, fin_zero(n), col)
}

/// The first-row Laplace expansion, relative to a determinant on minors.
define matrix_laplace_first_row[R: Ring](n: Nat, det_small: Matrix[R, n, n] -> R, a: Matrix[R, n.suc, n.suc]) -> R {
    matrix_laplace_row[R](n, det_small, a, fin_zero(n))
}

/// The zero function on square matrices of a fixed size.
define matrix_zero_det_function[R: Ring](n: Nat, a: Matrix[R, n, n]) -> R {
    R.0
}

/// The position of an item in a list, or zero if it is absent.
define list_index_or_zero[I](items: List[I], item: I) -> Nat {
    match items {
        List.nil {
            Nat.0
        }
        List.cons(head, tail) {
            if head = item {
                Nat.0
            } else {
                list_index_or_zero[I](tail, item).suc
            }
        }
    }
}

/// A determinant expansion over explicit row and column lists with a supplied entry function.
define determinant_list[R: Ring, I](entry: (I, I) -> R, rows: List[I], cols: List[I]) -> R {
    match rows {
        List.nil {
            R.1
        }
        List.cons(row, tail_rows) {
            sum[R](map[I, R](cols, function(col: I) {
                entry(row, col) *
                    alternating_sign[R](list_index_or_zero[I](cols, col)) *
                    determinant_list[R, I](entry, tail_rows, cols.remove_one(col))
            }))
        }
    }
}

/// The entry function of a square matrix as a binary function on its finite index set.
define square_matrix_entry_function[A](n: Nat, a: Matrix[A, n, n], row: Fin[n], col: Fin[n]) -> A {
    matrix_entry[A](n, n, a, row, col)
}

/// A determinant expansion over explicit row and column index lists in a fixed ambient matrix.
define matrix_det_list[R: Ring](n: Nat, a: Matrix[R, n, n], rows: List[Fin[n]], cols: List[Fin[n]]) -> R {
    determinant_list[R, Fin[n]](square_matrix_entry_function[R](n, a), rows, cols)
}

attributes Matrix[A, m: Nat, n: Nat] {
    /// The entry at row `i` and column `j`.
    define at(self, i: Fin[m], j: Fin[n]) -> A {
        matrix_entry[A](m, n, self, i, j)
    }

    /// The transpose of this matrix.
    define transpose(self) -> Matrix[A, n, m] {
        matrix_transpose[A](m, n, self)
    }

    /// The `i`th row of this matrix.
    define row(self, i: Fin[m], j: Fin[n]) -> A {
        matrix_row[A](m, n, self, i, j)
    }

    /// The `j`th column of this matrix.
    define col(self, j: Fin[n], i: Fin[m]) -> A {
        matrix_col[A](m, n, self, j, i)
    }
}

attributes Matrix[A: Zero, m: Nat, n: Nat] {
    /// The zero matrix.
    let zero: Matrix[A, m, n] = matrix_zero[A](m, n)
}

attributes Matrix[A: Add, m: Nat, n: Nat] {
    /// The pointwise sum of two matrices.
    define add(self, other: Matrix[A, m, n]) -> Matrix[A, m, n] {
        matrix_add[A](m, n, self, other)
    }
}

attributes Matrix[A: Neg, m: Nat, n: Nat] {
    /// The pointwise additive inverse of a matrix.
    define neg(self) -> Matrix[A, m, n] {
        matrix_neg[A](m, n, self)
    }
}

attributes Matrix[A: AddGroup, m: Nat, n: Nat] {
    /// The pointwise difference of two matrices.
    define sub(self, other: Matrix[A, m, n]) -> Matrix[A, m, n] {
        matrix_sub[A](m, n, self, other)
    }
}

attributes Matrix[S: Semiring, m: Nat, n: Nat] {
    /// Matrix multiplication with a compatible right factor.
    define mul(self, p: Nat, other: Matrix[S, n, p]) -> Matrix[S, m, p] {
        matrix_mul[S](m, n, p, self, other)
    }
}

/// Matrices have pointwise addition.
instance Matrix[A: Add, m: Nat, n: Nat]: Add {
    let add = Matrix[A, m, n].add
}

/// Matrices have a zero element when their entries do.
instance Matrix[A: Zero, m: Nat, n: Nat]: Zero {
    let 0 = Matrix[A, m, n].zero
}

/// Matrices have pointwise additive inverses.
instance Matrix[A: Neg, m: Nat, n: Nat]: Neg {
    let neg = Matrix[A, m, n].neg
}

/// Matrix entry agrees with the underlying entry function.
theorem matrix_entry_self[A](m: Nat, n: Nat, a: Matrix[A, m, n], i: Fin[m], j: Fin[n]) {
    matrix_entry[A](m, n, a, i, j) = a.entry(i, j)
}

/// A matrix is determined by its entries.
theorem matrix_ext[A](m: Nat, n: Nat, a: Matrix[A, m, n], b: Matrix[A, m, n]) {
    (forall(i: Fin[m], j: Fin[n]) { matrix_entry[A](m, n, a, i, j) = matrix_entry[A](m, n, b, i, j) }) implies a = b
}

/// The zero matrix has zero in every entry.
theorem matrix_entry_zero[A: Zero](m: Nat, n: Nat, i: Fin[m], j: Fin[n]) {
    matrix_entry[A](m, n, matrix_zero[A](m, n), i, j) = A.0
}

/// Pointwise matrix addition agrees with addition of entries.
theorem matrix_entry_add[A: Add](m: Nat, n: Nat, a: Matrix[A, m, n], b: Matrix[A, m, n], i: Fin[m], j: Fin[n]) {
    matrix_entry[A](m, n, a + b, i, j) = matrix_entry[A](m, n, a, i, j) + matrix_entry[A](m, n, b, i, j)
} by {
    matrix_entry[A](m, n, a + b, i, j) = matrix_add_entry[A](m, n, a, b, i, j)
}

/// Pointwise matrix negation agrees with negation of entries.
theorem matrix_entry_neg[A: Neg](m: Nat, n: Nat, a: Matrix[A, m, n], i: Fin[m], j: Fin[n]) {
    matrix_entry[A](m, n, -a, i, j) = -matrix_entry[A](m, n, a, i, j)
} by {
    let lhs = -a
    matrix_entry[A](m, n, lhs, i, j) = matrix_neg_entry[A](m, n, a, i, j)
}

/// Pointwise matrix subtraction agrees with subtraction of entries.
theorem matrix_entry_sub[A: AddGroup](m: Nat, n: Nat, a: Matrix[A, m, n], b: Matrix[A, m, n], i: Fin[m], j: Fin[n]) {
    matrix_entry[A](m, n, matrix_sub[A](m, n, a, b), i, j) = matrix_entry[A](m, n, a, i, j) - matrix_entry[A](m, n, b, i, j)
} by {
    matrix_sub_entry[A](m, n, a, b, i, j) = matrix_entry[A](m, n, a, i, j) - matrix_entry[A](m, n, b, i, j)
}

/// Matrix scalar multiplication agrees with scalar multiplication of entries.
theorem matrix_entry_smul[R: Ring, M: AddCommGroup](
    carrier: Module[R, M], m: Nat, n: Nat, r: R, a: Matrix[M, m, n], i: Fin[m], j: Fin[n]
) {
    matrix_entry[M](m, n, matrix_smul[R, M](carrier, m, n, r, a), i, j) = carrier.smul(r, matrix_entry[M](m, n, a, i, j))
} by {
    matrix_smul_entry[R, M](carrier, m, n, r, a, i, j) = carrier.smul(r, matrix_entry[M](m, n, a, i, j))
}

/// Matrix transposition swaps indices.
theorem matrix_entry_transpose[A](m: Nat, n: Nat, a: Matrix[A, m, n], i: Fin[m], j: Fin[n]) {
    matrix_entry[A](n, m, matrix_transpose[A](m, n, a), j, i) = matrix_entry[A](m, n, a, i, j)
} by {
    matrix_entry[A](n, m, matrix_transpose[A](m, n, a), j, i) = matrix_transpose_entry[A](m, n, a, j, i)
    matrix_transpose_entry[A](m, n, a, j, i) = matrix_entry[A](m, n, a, i, j)
}

/// Matrix rows agree with the corresponding entries.
theorem matrix_row_entry[A](m: Nat, n: Nat, a: Matrix[A, m, n], i: Fin[m], j: Fin[n]) {
    matrix_row[A](m, n, a, i, j) = matrix_entry[A](m, n, a, i, j)
}

/// Matrix columns agree with the corresponding entries.
theorem matrix_col_entry[A](m: Nat, n: Nat, a: Matrix[A, m, n], j: Fin[n], i: Fin[m]) {
    matrix_col[A](m, n, a, j, i) = matrix_entry[A](m, n, a, i, j)
}

/// A matrix product entry is the finite sum of its multiplication terms.
theorem matrix_entry_mul[S: Semiring](m: Nat, n: Nat, p: Nat, a: Matrix[S, m, n], b: Matrix[S, n, p], i: Fin[m], j: Fin[p]) {
    matrix_entry[S](m, p, matrix_mul[S](m, n, p, a, b), i, j) = fin_sum[S](n, matrix_mul_term[S](m, n, p, a, b, i, j))
} by {
    matrix_entry[S](m, p, matrix_mul[S](m, n, p, a, b), i, j) = matrix_mul_entry[S](m, n, p, a, b, i, j)
    matrix_mul_entry[S](m, n, p, a, b, i, j) = fin_sum[S](n, matrix_mul_term[S](m, n, p, a, b, i, j))
}

/// A diagonal entry of the identity matrix is one.
theorem matrix_entry_one_diag[S: Semiring](n: Nat, i: Fin[n]) {
    matrix_entry[S](n, n, matrix_one[S](n), i, i) = S.1
} by {
    matrix_entry[S](n, n, matrix_one[S](n), i, i) = matrix_one_entry[S](n, i, i)
    matrix_one_entry[S](n, i, i) = S.1
}

/// An off-diagonal entry of the identity matrix is zero.
theorem matrix_entry_one_off_diag[S: Semiring](n: Nat, i: Fin[n], j: Fin[n]) {
    i != j implies matrix_entry[S](n, n, matrix_one[S](n), i, j) = S.0
} by {
    if i != j {
        matrix_entry[S](n, n, matrix_one[S](n), i, j) = matrix_one_entry[S](n, i, j)
        matrix_one_entry[S](n, i, j) = S.0
    }
}

/// Transposition preserves the identity matrix.
theorem matrix_transpose_one[S: Semiring](n: Nat) {
    matrix_transpose[S](n, n, matrix_one[S](n)) = matrix_one[S](n)
} by {
    let lhs = matrix_transpose[S](n, n, matrix_one[S](n))
    let rhs = matrix_one[S](n)
    forall(i: Fin[n], j: Fin[n]) {
        matrix_entry_transpose[S](n, n, matrix_one[S](n), j, i)
        if i = j {
            matrix_entry_one_diag[S](n, i)
            matrix_entry_one_diag[S](n, j)
            matrix_entry[S](n, n, lhs, i, j) = S.1
            matrix_entry[S](n, n, rhs, i, j) = S.1
            matrix_entry[S](n, n, lhs, i, j) = matrix_entry[S](n, n, rhs, i, j)
        } else {
            j != i
            matrix_entry_one_off_diag[S](n, j, i)
            matrix_entry_one_off_diag[S](n, i, j)
            matrix_entry[S](n, n, lhs, i, j) = S.0
            matrix_entry[S](n, n, rhs, i, j) = S.0
            matrix_entry[S](n, n, lhs, i, j) = matrix_entry[S](n, n, rhs, i, j)
        }
    }
    matrix_ext[S](n, n, lhs, rhs)
    lhs = rhs
}

/// Multiplying on the right by the identity matrix changes nothing.
theorem matrix_mul_one_right[S: Semiring](m: Nat, n: Nat, a: Matrix[S, m, n]) {
    matrix_mul[S](m, n, n, a, matrix_one[S](n)) = a
} by {
    let lhs = matrix_mul[S](m, n, n, a, matrix_one[S](n))
    forall(i: Fin[m], j: Fin[n]) {
        let term = matrix_mul_term[S](m, n, n, a, matrix_one[S](n), i, j)
        forall(k: Fin[n]) {
            if k != j {
                matrix_entry_one_off_diag[S](n, k, j)
                term(k) = matrix_entry[S](m, n, a, i, k) * matrix_entry[S](n, n, matrix_one[S](n), k, j)
                matrix_entry[S](n, n, matrix_one[S](n), k, j) = S.0
                term(k) = matrix_entry[S](m, n, a, i, k) * S.0
                matrix_entry[S](m, n, a, i, k) * S.0 = S.0
                term(k) = S.0
            }
        }
        fin_sum_single_nonzero[S](n, term, j)
        matrix_entry_mul[S](m, n, n, a, matrix_one[S](n), i, j)
        matrix_entry[S](m, n, lhs, i, j) = fin_sum[S](n, term)
        fin_sum[S](n, term) = term(j)
        term(j) = matrix_entry[S](m, n, a, i, j) * matrix_entry[S](n, n, matrix_one[S](n), j, j)
        matrix_entry_one_diag[S](n, j)
        matrix_entry[S](n, n, matrix_one[S](n), j, j) = S.1
        term(j) = matrix_entry[S](m, n, a, i, j) * S.1
        matrix_entry[S](m, n, a, i, j) * S.1 = matrix_entry[S](m, n, a, i, j)
        matrix_entry[S](m, n, lhs, i, j) = matrix_entry[S](m, n, a, i, j)
    }
    matrix_ext[S](m, n, lhs, a)
    lhs = a
}

/// Multiplying on the left by the identity matrix changes nothing.
theorem matrix_mul_one_left[S: Semiring](m: Nat, n: Nat, a: Matrix[S, m, n]) {
    matrix_mul[S](m, m, n, matrix_one[S](m), a) = a
} by {
    let lhs = matrix_mul[S](m, m, n, matrix_one[S](m), a)
    forall(i: Fin[m], j: Fin[n]) {
        let term = matrix_mul_term[S](m, m, n, matrix_one[S](m), a, i, j)
        forall(k: Fin[m]) {
            if k != i {
                if i = k {
                    false
                }
                i != k
                matrix_entry_one_off_diag[S](m, i, k)
                term(k) = matrix_entry[S](m, m, matrix_one[S](m), i, k) * matrix_entry[S](m, n, a, k, j)
                matrix_entry[S](m, m, matrix_one[S](m), i, k) = S.0
                term(k) = S.0 * matrix_entry[S](m, n, a, k, j)
                S.0 * matrix_entry[S](m, n, a, k, j) = S.0
                term(k) = S.0
            }
        }
        fin_sum_single_nonzero[S](m, term, i)
        matrix_entry_mul[S](m, m, n, matrix_one[S](m), a, i, j)
        matrix_entry[S](m, n, lhs, i, j) = fin_sum[S](m, term)
        fin_sum[S](m, term) = term(i)
        term(i) = matrix_entry[S](m, m, matrix_one[S](m), i, i) * matrix_entry[S](m, n, a, i, j)
        matrix_entry_one_diag[S](m, i)
        matrix_entry[S](m, m, matrix_one[S](m), i, i) = S.1
        term(i) = S.1 * matrix_entry[S](m, n, a, i, j)
        S.1 * matrix_entry[S](m, n, a, i, j) = matrix_entry[S](m, n, a, i, j)
        matrix_entry[S](m, n, lhs, i, j) = matrix_entry[S](m, n, a, i, j)
    }
    matrix_ext[S](m, n, lhs, a)
    lhs = a
}

/// A square matrix product entry is the finite sum of its multiplication terms.
theorem matrix_entry_square_mul[S: Semiring](n: Nat, a: Matrix[S, n, n], b: Matrix[S, n, n], i: Fin[n], j: Fin[n]) {
    matrix_entry[S](n, n, square_matrix_mul[S](n, a, b), i, j) = fin_sum[S](n, matrix_mul_term[S](n, n, n, a, b, i, j))
} by {
    matrix_entry_mul[S](n, n, n, a, b, i, j)
}

/// A minor entry is the corresponding entry with the selected row and column skipped.
theorem matrix_entry_minor[A](n: Nat, a: Matrix[A, n.suc, n.suc], row: Fin[n.suc], col: Fin[n.suc], i: Fin[n], j: Fin[n]) {
    matrix_entry[A](n, n, matrix_minor[A](n, a, row, col), i, j) =
        matrix_entry[A](n.suc, n.suc, a, fin_skip(n, row, i), fin_skip(n, col, j))
} by {
    matrix_entry[A](n, n, matrix_minor[A](n, a, row, col), i, j) = matrix_minor_entry[A](n, a, row, col, i, j)
    matrix_minor_entry[A](n, a, row, col, i, j) =
        matrix_entry[A](n.suc, n.suc, a, fin_skip(n, row, i), fin_skip(n, col, j))
}

/// The minor of the zero matrix is the zero matrix.
theorem matrix_minor_zero[A: Zero](n: Nat, row: Fin[n.suc], col: Fin[n.suc]) {
    matrix_minor[A](n, matrix_zero[A](n.suc, n.suc), row, col) = matrix_zero[A](n, n)
} by {
    let lhs = matrix_minor[A](n, matrix_zero[A](n.suc, n.suc), row, col)
    let rhs = matrix_zero[A](n, n)
    forall(i: Fin[n], j: Fin[n]) {
        matrix_entry_minor[A](n, matrix_zero[A](n.suc, n.suc), row, col, i, j)
        matrix_entry_zero[A](n.suc, n.suc, fin_skip(n, row, i), fin_skip(n, col, j))
        matrix_entry_zero[A](n, n, i, j)
        matrix_entry[A](n, n, lhs, i, j) = A.0
        matrix_entry[A](n, n, rhs, i, j) = A.0
        matrix_entry[A](n, n, lhs, i, j) = matrix_entry[A](n, n, rhs, i, j)
    }
    matrix_ext[A](n, n, lhs, rhs)
    lhs = rhs
}

/// Minors commute with pointwise matrix addition.
theorem matrix_minor_add[A: Add](n: Nat, a: Matrix[A, n.suc, n.suc], b: Matrix[A, n.suc, n.suc], row: Fin[n.suc], col: Fin[n.suc]) {
    matrix_minor[A](n, a + b, row, col) = matrix_minor[A](n, a, row, col) + matrix_minor[A](n, b, row, col)
} by {
    let lhs = matrix_minor[A](n, a + b, row, col)
    let ma = matrix_minor[A](n, a, row, col)
    let mb = matrix_minor[A](n, b, row, col)
    let rhs = ma + mb
    forall(i: Fin[n], j: Fin[n]) {
        matrix_entry_minor[A](n, a + b, row, col, i, j)
        matrix_entry_add[A](n.suc, n.suc, a, b, fin_skip(n, row, i), fin_skip(n, col, j))
        matrix_entry_minor[A](n, a, row, col, i, j)
        matrix_entry_minor[A](n, b, row, col, i, j)
        matrix_entry_add[A](n, n, ma, mb, i, j)
        matrix_entry[A](n, n, lhs, i, j) = matrix_entry[A](n, n, ma, i, j) + matrix_entry[A](n, n, mb, i, j)
        matrix_entry[A](n, n, rhs, i, j) = matrix_entry[A](n, n, ma, i, j) + matrix_entry[A](n, n, mb, i, j)
        matrix_entry[A](n, n, lhs, i, j) = matrix_entry[A](n, n, rhs, i, j)
    }
    matrix_ext[A](n, n, lhs, rhs)
    lhs = rhs
}

/// Minors commute with pointwise negation.
theorem matrix_minor_neg[A: Neg](n: Nat, a: Matrix[A, n.suc, n.suc], row: Fin[n.suc], col: Fin[n.suc]) {
    matrix_minor[A](n, -a, row, col) = -matrix_minor[A](n, a, row, col)
} by {
    let lhs = matrix_minor[A](n, -a, row, col)
    let ma = matrix_minor[A](n, a, row, col)
    let rhs = -ma
    forall(i: Fin[n], j: Fin[n]) {
        matrix_entry_minor[A](n, -a, row, col, i, j)
        matrix_entry_neg[A](n.suc, n.suc, a, fin_skip(n, row, i), fin_skip(n, col, j))
        matrix_entry_minor[A](n, a, row, col, i, j)
        matrix_entry_neg[A](n, n, ma, i, j)
        matrix_entry[A](n, n, lhs, i, j) = -matrix_entry[A](n, n, ma, i, j)
        matrix_entry[A](n, n, rhs, i, j) = -matrix_entry[A](n, n, ma, i, j)
        matrix_entry[A](n, n, lhs, i, j) = matrix_entry[A](n, n, rhs, i, j)
    }
    matrix_ext[A](n, n, lhs, rhs)
    lhs = rhs
}

/// Minors commute with pointwise subtraction.
theorem matrix_minor_sub[A: AddGroup](n: Nat, a: Matrix[A, n.suc, n.suc], b: Matrix[A, n.suc, n.suc], row: Fin[n.suc], col: Fin[n.suc]) {
    matrix_minor[A](n, matrix_sub[A](n.suc, n.suc, a, b), row, col) = matrix_sub[A](n, n, matrix_minor[A](n, a, row, col), matrix_minor[A](n, b, row, col))
} by {
    let lhs = matrix_minor[A](n, matrix_sub[A](n.suc, n.suc, a, b), row, col)
    let ma = matrix_minor[A](n, a, row, col)
    let mb = matrix_minor[A](n, b, row, col)
    let rhs = matrix_sub[A](n, n, ma, mb)
    forall(i: Fin[n], j: Fin[n]) {
        matrix_entry_minor[A](n, matrix_sub[A](n.suc, n.suc, a, b), row, col, i, j)
        matrix_entry_sub[A](n.suc, n.suc, a, b, fin_skip(n, row, i), fin_skip(n, col, j))
        matrix_entry_minor[A](n, a, row, col, i, j)
        matrix_entry_minor[A](n, b, row, col, i, j)
        matrix_entry_sub[A](n, n, ma, mb, i, j)
        matrix_entry[A](n, n, lhs, i, j) = matrix_entry[A](n, n, ma, i, j) - matrix_entry[A](n, n, mb, i, j)
        matrix_entry[A](n, n, rhs, i, j) = matrix_entry[A](n, n, ma, i, j) - matrix_entry[A](n, n, mb, i, j)
        matrix_entry[A](n, n, lhs, i, j) = matrix_entry[A](n, n, rhs, i, j)
    }
    matrix_ext[A](n, n, lhs, rhs)
    lhs = rhs
}

/// Minors commute with entrywise scalar multiplication.
theorem matrix_minor_smul[R: Ring, M: AddCommGroup](carrier: Module[R, M], n: Nat, r: R, a: Matrix[M, n.suc, n.suc], row: Fin[n.suc], col: Fin[n.suc]) {
    matrix_minor[M](n, matrix_smul[R, M](carrier, n.suc, n.suc, r, a), row, col) =
        matrix_smul[R, M](carrier, n, n, r, matrix_minor[M](n, a, row, col))
} by {
    let lhs = matrix_minor[M](n, matrix_smul[R, M](carrier, n.suc, n.suc, r, a), row, col)
    let ma = matrix_minor[M](n, a, row, col)
    let rhs = matrix_smul[R, M](carrier, n, n, r, ma)
    forall(i: Fin[n], j: Fin[n]) {
        matrix_entry_minor[M](n, matrix_smul[R, M](carrier, n.suc, n.suc, r, a), row, col, i, j)
        matrix_entry_smul[R, M](carrier, n.suc, n.suc, r, a, fin_skip(n, row, i), fin_skip(n, col, j))
        matrix_entry_minor[M](n, a, row, col, i, j)
        matrix_entry_smul[R, M](carrier, n, n, r, ma, i, j)
        matrix_entry[M](n, n, lhs, i, j) = carrier.smul(r, matrix_entry[M](n, n, ma, i, j))
        matrix_entry[M](n, n, rhs, i, j) = carrier.smul(r, matrix_entry[M](n, n, ma, i, j))
        matrix_entry[M](n, n, lhs, i, j) = matrix_entry[M](n, n, rhs, i, j)
    }
    matrix_ext[M](n, n, lhs, rhs)
    lhs = rhs
}

/// A minor of a transpose is the transpose of the swapped minor.
theorem matrix_minor_transpose[A](n: Nat, a: Matrix[A, n.suc, n.suc], row: Fin[n.suc], col: Fin[n.suc]) {
    matrix_minor[A](n, matrix_transpose[A](n.suc, n.suc, a), row, col) = matrix_transpose[A](n, n, matrix_minor[A](n, a, col, row))
} by {
    let lhs = matrix_minor[A](n, matrix_transpose[A](n.suc, n.suc, a), row, col)
    let ma = matrix_minor[A](n, a, col, row)
    let rhs = matrix_transpose[A](n, n, ma)
    forall(i: Fin[n], j: Fin[n]) {
        matrix_entry_minor[A](n, matrix_transpose[A](n.suc, n.suc, a), row, col, i, j)
        matrix_entry_transpose[A](n.suc, n.suc, a, fin_skip(n, col, j), fin_skip(n, row, i))
        matrix_entry_minor[A](n, a, col, row, j, i)
        matrix_entry_transpose[A](n, n, ma, j, i)
        matrix_entry[A](n, n, lhs, i, j) = matrix_entry[A](n.suc, n.suc, a, fin_skip(n, col, j), fin_skip(n, row, i))
        matrix_entry[A](n, n, rhs, i, j) = matrix_entry[A](n.suc, n.suc, a, fin_skip(n, col, j), fin_skip(n, row, i))
        matrix_entry[A](n, n, lhs, i, j) = matrix_entry[A](n, n, rhs, i, j)
    }
    matrix_ext[A](n, n, lhs, rhs)
    lhs = rhs
}

/// Cofactors with a supplied minor determinant unfold to sign times minor determinant.
theorem matrix_cofactor_with_eq[R: Ring](n: Nat, det_small: Matrix[R, n, n] -> R, a: Matrix[R, n.suc, n.suc], row: Fin[n.suc], col: Fin[n.suc]) {
    matrix_cofactor_with[R](n, det_small, a, row, col) =
        matrix_cofactor_sign[R](n.suc, row, col) * det_small(matrix_minor[R](n, a, row, col))
}

/// A Laplace row summand unfolds to an entry times its supplied cofactor.
theorem matrix_laplace_row_term_eq[R: Ring](n: Nat, det_small: Matrix[R, n, n] -> R, a: Matrix[R, n.suc, n.suc], row: Fin[n.suc], col: Fin[n.suc]) {
    matrix_laplace_row_term[R](n, det_small, a, row, col) =
        matrix_entry[R](n.suc, n.suc, a, row, col) * matrix_cofactor_with[R](n, det_small, a, row, col)
}

/// The first-row Laplace expansion is the row expansion at the first index.
theorem matrix_laplace_first_row_eq_row[R: Ring](n: Nat, det_small: Matrix[R, n, n] -> R, a: Matrix[R, n.suc, n.suc]) {
    matrix_laplace_first_row[R](n, det_small, a) = matrix_laplace_row[R](n, det_small, a, fin_zero(n))
}

/// Cofactors vanish when the supplied minor determinant is the zero function.
theorem matrix_cofactor_with_zero_det[R: Ring](n: Nat, a: Matrix[R, n.suc, n.suc], row: Fin[n.suc], col: Fin[n.suc]) {
    matrix_cofactor_with[R](n, matrix_zero_det_function[R](n), a, row, col) = R.0
} by {
    matrix_cofactor_with_eq[R](n, matrix_zero_det_function[R](n), a, row, col)
    matrix_zero_det_function[R](n, matrix_minor[R](n, a, row, col)) = R.0
    matrix_cofactor_with[R](n, matrix_zero_det_function[R](n), a, row, col) = matrix_cofactor_sign[R](n.suc, row, col) * R.0
    matrix_cofactor_sign[R](n.suc, row, col) * R.0 = R.0
}

/// A Laplace summand vanishes when the supplied minor determinant is the zero function.
theorem matrix_laplace_row_term_zero_det[R: Ring](n: Nat, a: Matrix[R, n.suc, n.suc], row: Fin[n.suc], col: Fin[n.suc]) {
    matrix_laplace_row_term[R](n, matrix_zero_det_function[R](n), a, row, col) = R.0
} by {
    matrix_laplace_row_term_eq[R](n, matrix_zero_det_function[R](n), a, row, col)
    matrix_cofactor_with_zero_det[R](n, a, row, col)
    matrix_laplace_row_term[R](n, matrix_zero_det_function[R](n), a, row, col) = matrix_entry[R](n.suc, n.suc, a, row, col) * R.0
    matrix_entry[R](n.suc, n.suc, a, row, col) * R.0 = R.0
}

/// A Laplace row expansion vanishes when the supplied minor determinant is the zero function.
theorem matrix_laplace_row_zero_det[R: Ring](n: Nat, a: Matrix[R, n.suc, n.suc], row: Fin[n.suc]) {
    matrix_laplace_row[R](n, matrix_zero_det_function[R](n), a, row) = R.0
} by {
    forall(col: Fin[n.suc]) {
        matrix_laplace_row_term_zero_det[R](n, a, row, col)
        matrix_laplace_row_term[R](n, matrix_zero_det_function[R](n), a, row, col) = fin_zero_function[R](n.suc, col)
    }
    fin_sum_pointwise_eq[R](n.suc, matrix_laplace_row_term[R](n, matrix_zero_det_function[R](n), a, row), fin_zero_function[R](n.suc))
    fin_sum_zero_function[R](n.suc)
}

/// A first-row Laplace expansion vanishes when the supplied minor determinant is the zero function.
theorem matrix_laplace_first_row_zero_det[R: Ring](n: Nat, a: Matrix[R, n.suc, n.suc]) {
    matrix_laplace_first_row[R](n, matrix_zero_det_function[R](n), a) = R.0
} by {
    matrix_laplace_first_row_eq_row[R](n, matrix_zero_det_function[R](n), a)
    matrix_laplace_row_zero_det[R](n, a, fin_zero(n))
}

/// A Laplace summand of the zero matrix is zero.
theorem matrix_laplace_row_term_zero_matrix[R: Ring](n: Nat, det_small: Matrix[R, n, n] -> R, row: Fin[n.suc], col: Fin[n.suc]) {
    matrix_laplace_row_term[R](n, det_small, matrix_zero[R](n.suc, n.suc), row, col) = R.0
} by {
    matrix_laplace_row_term_eq[R](n, det_small, matrix_zero[R](n.suc, n.suc), row, col)
    matrix_entry_zero[R](n.suc, n.suc, row, col)
    matrix_laplace_row_term[R](n, det_small, matrix_zero[R](n.suc, n.suc), row, col) = R.0 * matrix_cofactor_with[R](n, det_small, matrix_zero[R](n.suc, n.suc), row, col)
    R.0 * matrix_cofactor_with[R](n, det_small, matrix_zero[R](n.suc, n.suc), row, col) = R.0
}

/// A Laplace row expansion of the zero matrix is zero.
theorem matrix_laplace_row_zero_matrix[R: Ring](n: Nat, det_small: Matrix[R, n, n] -> R, row: Fin[n.suc]) {
    matrix_laplace_row[R](n, det_small, matrix_zero[R](n.suc, n.suc), row) = R.0
} by {
    forall(col: Fin[n.suc]) {
        matrix_laplace_row_term_zero_matrix[R](n, det_small, row, col)
        matrix_laplace_row_term[R](n, det_small, matrix_zero[R](n.suc, n.suc), row, col) = fin_zero_function[R](n.suc, col)
    }
    fin_sum_pointwise_eq[R](n.suc, matrix_laplace_row_term[R](n, det_small, matrix_zero[R](n.suc, n.suc), row), fin_zero_function[R](n.suc))
    fin_sum_zero_function[R](n.suc)
}

/// The first-row Laplace expansion of the zero matrix is zero.
theorem matrix_laplace_first_row_zero_matrix[R: Ring](n: Nat, det_small: Matrix[R, n, n] -> R) {
    matrix_laplace_first_row[R](n, det_small, matrix_zero[R](n.suc, n.suc)) = R.0
} by {
    matrix_laplace_first_row_eq_row[R](n, det_small, matrix_zero[R](n.suc, n.suc))
    matrix_laplace_row_zero_matrix[R](n, det_small, fin_zero(n))
}

/// The determinant-list expansion over no rows is one.
theorem determinant_list_nil_rows[R: Ring, I](entry: (I, I) -> R, cols: List[I]) {
    determinant_list[R, I](entry, List.nil[I], cols) = R.1
}

/// The index of any item in the empty list defaults to zero.
theorem list_index_or_zero_nil[I](item: I) {
    list_index_or_zero[I](List.nil[I], item) = Nat.0
}

/// The head of a list has index zero.
theorem list_index_or_zero_cons_self[I](head: I, tail: List[I]) {
    list_index_or_zero[I](List.cons(head, tail), head) = Nat.0
}

/// An item different from the head has successor index in the tail.
theorem list_index_or_zero_cons_other[I](head: I, tail: List[I], item: I) {
    head != item implies list_index_or_zero[I](List.cons(head, tail), item) = list_index_or_zero[I](tail, item).suc
} by {
    if head != item {
        list_index_or_zero[I](List.cons(head, tail), item) = list_index_or_zero[I](tail, item).suc
    }
}

/// The determinant-list expansion over a cons row list unfolds along its head.
theorem determinant_list_cons_rows[R: Ring, I](entry: (I, I) -> R, row: I, tail_rows: List[I], cols: List[I]) {
    determinant_list[R, I](entry, List.cons(row, tail_rows), cols) =
        sum[R](map[I, R](cols, function(col: I) {
            entry(row, col) *
                alternating_sign[R](list_index_or_zero[I](cols, col)) *
                determinant_list[R, I](entry, tail_rows, cols.remove_one(col))
        }))
}

/// Matrix determinant-list expansion over no rows is one.
theorem matrix_det_list_nil_rows[R: Ring](n: Nat, a: Matrix[R, n, n], cols: List[Fin[n]]) {
    matrix_det_list[R](n, a, List.nil[Fin[n]], cols) = R.1
} by {
    determinant_list_nil_rows[R, Fin[n]](square_matrix_entry_function[R](n, a), cols)
}

/// The matrix determinant-list expansion over a cons row list unfolds along its head.
theorem matrix_det_list_cons_rows[R: Ring](n: Nat, a: Matrix[R, n, n], row: Fin[n], tail_rows: List[Fin[n]], cols: List[Fin[n]]) {
    matrix_det_list[R](n, a, List.cons(row, tail_rows), cols) =
        sum[R](map[Fin[n], R](cols, function(col: Fin[n]) {
            square_matrix_entry_function[R](n, a)(row, col) *
                alternating_sign[R](list_index_or_zero[Fin[n]](cols, col)) *
                determinant_list[R, Fin[n]](square_matrix_entry_function[R](n, a), tail_rows, cols.remove_one(col))
        }))
} by {
    determinant_list_cons_rows[R, Fin[n]](square_matrix_entry_function[R](n, a), row, tail_rows, cols)
}

/// A zero inner dimension gives the zero product entry.
theorem matrix_mul_inner_zero[S: Semiring](m: Nat, n: Nat, p: Nat, a: Matrix[S, m, n], b: Matrix[S, n, p], i: Fin[m], j: Fin[p]) {
    n = Nat.0 implies matrix_entry[S](m, p, matrix_mul[S](m, n, p, a, b), i, j) = S.0
} by {
    if n = Nat.0 {
        matrix_entry_mul[S](m, n, p, a, b, i, j)
        fin_sum_zero_of_bound_zero[S](n, matrix_mul_term[S](m, n, p, a, b, i, j))
    }
}

/// If the selected row of the left factor is zero, the product entry is zero.
theorem matrix_mul_zero_left_row[S: Semiring](m: Nat, n: Nat, p: Nat, a: Matrix[S, m, n], b: Matrix[S, n, p], i: Fin[m], j: Fin[p]) {
    (forall(k: Fin[n]) { matrix_entry[S](m, n, a, i, k) = S.0 }) implies matrix_entry[S](m, p, matrix_mul[S](m, n, p, a, b), i, j) = S.0
} by {
    let prod = matrix_mul[S](m, n, p, a, b)
    if forall(k: Fin[n]) { matrix_entry[S](m, n, a, i, k) = S.0 } {
        forall(k: Fin[n]) {
            matrix_mul_term[S](m, n, p, a, b, i, j, k) = matrix_entry[S](m, n, a, i, k) * matrix_entry[S](n, p, b, k, j)
            matrix_entry[S](m, n, a, i, k) = S.0
            S.0 * matrix_entry[S](n, p, b, k, j) = S.0
            matrix_mul_term[S](m, n, p, a, b, i, j, k) = fin_zero_function[S](n, k)
        }
        fin_sum_pointwise_eq[S](n, matrix_mul_term[S](m, n, p, a, b, i, j), fin_zero_function[S](n))
        matrix_entry_mul[S](m, n, p, a, b, i, j)
        fin_sum[S](n, matrix_mul_term[S](m, n, p, a, b, i, j)) = fin_sum[S](n, fin_zero_function[S](n))
        fin_sum_zero_function[S](n)
        fin_sum[S](n, fin_zero_function[S](n)) = S.0
        matrix_entry[S](m, p, prod, i, j) = S.0
    }
}

/// If the selected column of the right factor is zero, the product entry is zero.
theorem matrix_mul_zero_right_col[S: Semiring](m: Nat, n: Nat, p: Nat, a: Matrix[S, m, n], b: Matrix[S, n, p], i: Fin[m], j: Fin[p]) {
    (forall(k: Fin[n]) { matrix_entry[S](n, p, b, k, j) = S.0 }) implies matrix_entry[S](m, p, matrix_mul[S](m, n, p, a, b), i, j) = S.0
} by {
    let prod = matrix_mul[S](m, n, p, a, b)
    if forall(k: Fin[n]) { matrix_entry[S](n, p, b, k, j) = S.0 } {
        forall(k: Fin[n]) {
            matrix_mul_term[S](m, n, p, a, b, i, j, k) = matrix_entry[S](m, n, a, i, k) * matrix_entry[S](n, p, b, k, j)
            matrix_entry[S](n, p, b, k, j) = S.0
            matrix_entry[S](m, n, a, i, k) * S.0 = S.0
            matrix_mul_term[S](m, n, p, a, b, i, j, k) = fin_zero_function[S](n, k)
        }
        fin_sum_pointwise_eq[S](n, matrix_mul_term[S](m, n, p, a, b, i, j), fin_zero_function[S](n))
        matrix_entry_mul[S](m, n, p, a, b, i, j)
        fin_sum[S](n, matrix_mul_term[S](m, n, p, a, b, i, j)) = fin_sum[S](n, fin_zero_function[S](n))
        fin_sum_zero_function[S](n)
        fin_sum[S](n, fin_zero_function[S](n)) = S.0
        matrix_entry[S](m, p, prod, i, j) = S.0
    }
}

/// Multiplying by a zero matrix on the left gives a zero matrix.
theorem matrix_mul_zero_left[S: Semiring](m: Nat, n: Nat, p: Nat, b: Matrix[S, n, p]) {
    matrix_mul[S](m, n, p, matrix_zero[S](m, n), b) = matrix_zero[S](m, p)
} by {
    let z = matrix_zero[S](m, n)
    let lhs = matrix_mul[S](m, n, p, z, b)
    let rhs = matrix_zero[S](m, p)
    forall(i: Fin[m], j: Fin[p]) {
        forall(k: Fin[n]) {
            matrix_entry_zero[S](m, n, i, k)
            matrix_entry[S](m, n, z, i, k) = S.0
        }
        matrix_mul_zero_left_row[S](m, n, p, z, b, i, j)
        matrix_entry_zero[S](m, p, i, j)
        matrix_entry[S](m, p, lhs, i, j) = S.0
        matrix_entry[S](m, p, rhs, i, j) = S.0
        matrix_entry[S](m, p, lhs, i, j) = matrix_entry[S](m, p, rhs, i, j)
    }
    matrix_ext[S](m, p, lhs, rhs)
    lhs = rhs
}

/// Multiplying by a zero matrix on the right gives a zero matrix.
theorem matrix_mul_zero_right[S: Semiring](m: Nat, n: Nat, p: Nat, a: Matrix[S, m, n]) {
    matrix_mul[S](m, n, p, a, matrix_zero[S](n, p)) = matrix_zero[S](m, p)
} by {
    let z = matrix_zero[S](n, p)
    let lhs = matrix_mul[S](m, n, p, a, z)
    let rhs = matrix_zero[S](m, p)
    forall(i: Fin[m], j: Fin[p]) {
        forall(k: Fin[n]) {
            matrix_entry_zero[S](n, p, k, j)
            matrix_entry[S](n, p, z, k, j) = S.0
        }
        matrix_mul_zero_right_col[S](m, n, p, a, z, i, j)
        matrix_entry_zero[S](m, p, i, j)
        matrix_entry[S](m, p, lhs, i, j) = S.0
        matrix_entry[S](m, p, rhs, i, j) = S.0
        matrix_entry[S](m, p, lhs, i, j) = matrix_entry[S](m, p, rhs, i, j)
    }
    matrix_ext[S](m, p, lhs, rhs)
    lhs = rhs
}

/// Multiplying a square zero matrix on the left gives a zero matrix.
theorem square_matrix_mul_zero_left[S: Semiring](n: Nat, a: Matrix[S, n, n]) {
    square_matrix_mul[S](n, matrix_zero[S](n, n), a) = matrix_zero[S](n, n)
} by {
    matrix_mul_zero_left[S](n, n, n, a)
}

/// Multiplying a square zero matrix on the right gives a zero matrix.
theorem square_matrix_mul_zero_right[S: Semiring](n: Nat, a: Matrix[S, n, n]) {
    square_matrix_mul[S](n, a, matrix_zero[S](n, n)) = matrix_zero[S](n, n)
} by {
    matrix_mul_zero_right[S](n, n, n, a)
}

/// Matrix multiplication distributes over pointwise addition in the right factor.
theorem matrix_entry_mul_add_right[S: Semiring](
    m: Nat, n: Nat, p: Nat, a: Matrix[S, m, n], b: Matrix[S, n, p], c: Matrix[S, n, p], i: Fin[m], j: Fin[p]
) {
    matrix_entry[S](m, p, matrix_mul[S](m, n, p, a, b + c), i, j) =
        matrix_entry[S](m, p, matrix_mul[S](m, n, p, a, b), i, j) + matrix_entry[S](m, p, matrix_mul[S](m, n, p, a, c), i, j)
} by {
    let h = matrix_mul_term[S](m, n, p, a, b + c, i, j)
    let f = matrix_mul_term[S](m, n, p, a, b, i, j)
    let g = matrix_mul_term[S](m, n, p, a, c, i, j)
    forall(k: Fin[n]) {
        h(k) = matrix_entry[S](m, n, a, i, k) * matrix_entry[S](n, p, b + c, k, j)
        matrix_entry_add[S](n, p, b, c, k, j)
        matrix_entry[S](n, p, b + c, k, j) = matrix_entry[S](n, p, b, k, j) + matrix_entry[S](n, p, c, k, j)
        matrix_entry[S](m, n, a, i, k) * (matrix_entry[S](n, p, b, k, j) + matrix_entry[S](n, p, c, k, j)) =
            matrix_entry[S](m, n, a, i, k) * matrix_entry[S](n, p, b, k, j) + matrix_entry[S](m, n, a, i, k) * matrix_entry[S](n, p, c, k, j)
        f(k) = matrix_entry[S](m, n, a, i, k) * matrix_entry[S](n, p, b, k, j)
        g(k) = matrix_entry[S](m, n, a, i, k) * matrix_entry[S](n, p, c, k, j)
        fin_pointwise_add[S](n, f, g, k) = f(k) + g(k)
        h(k) = fin_pointwise_add[S](n, f, g, k)
    }
    function_extensionality(h, fin_pointwise_add[S](n, f, g))
    fin_sum_add[S](n, f, g)
    matrix_entry_mul[S](m, n, p, a, b + c, i, j)
    matrix_entry_mul[S](m, n, p, a, b, i, j)
    matrix_entry_mul[S](m, n, p, a, c, i, j)
}

/// Matrix multiplication distributes over addition in the right factor.
theorem matrix_mul_add_right[S: Semiring](
    m: Nat, n: Nat, p: Nat, a: Matrix[S, m, n], b: Matrix[S, n, p], c: Matrix[S, n, p]
) {
    matrix_mul[S](m, n, p, a, b + c) = matrix_mul[S](m, n, p, a, b) + matrix_mul[S](m, n, p, a, c)
} by {
    let lhs = matrix_mul[S](m, n, p, a, b + c)
    let ab = matrix_mul[S](m, n, p, a, b)
    let ac = matrix_mul[S](m, n, p, a, c)
    let rhs = ab + ac
    forall(i: Fin[m], j: Fin[p]) {
        matrix_entry_mul_add_right[S](m, n, p, a, b, c, i, j)
        matrix_entry_add[S](m, p, ab, ac, i, j)
        matrix_entry[S](m, p, lhs, i, j) = matrix_entry[S](m, p, ab, i, j) + matrix_entry[S](m, p, ac, i, j)
        matrix_entry[S](m, p, rhs, i, j) = matrix_entry[S](m, p, ab, i, j) + matrix_entry[S](m, p, ac, i, j)
        matrix_entry[S](m, p, lhs, i, j) = matrix_entry[S](m, p, rhs, i, j)
    }
    matrix_ext[S](m, p, lhs, rhs)
    lhs = rhs
}

/// Matrix multiplication distributes over pointwise addition in the left factor.
theorem matrix_entry_mul_add_left[S: Semiring](
    m: Nat, n: Nat, p: Nat, a: Matrix[S, m, n], b: Matrix[S, m, n], c: Matrix[S, n, p], i: Fin[m], j: Fin[p]
) {
    matrix_entry[S](m, p, matrix_mul[S](m, n, p, a + b, c), i, j) =
        matrix_entry[S](m, p, matrix_mul[S](m, n, p, a, c), i, j) + matrix_entry[S](m, p, matrix_mul[S](m, n, p, b, c), i, j)
} by {
    let h = matrix_mul_term[S](m, n, p, a + b, c, i, j)
    let f = matrix_mul_term[S](m, n, p, a, c, i, j)
    let g = matrix_mul_term[S](m, n, p, b, c, i, j)
    forall(k: Fin[n]) {
        h(k) = matrix_entry[S](m, n, a + b, i, k) * matrix_entry[S](n, p, c, k, j)
        matrix_entry_add[S](m, n, a, b, i, k)
        matrix_entry[S](m, n, a + b, i, k) = matrix_entry[S](m, n, a, i, k) + matrix_entry[S](m, n, b, i, k)
        (matrix_entry[S](m, n, a, i, k) + matrix_entry[S](m, n, b, i, k)) * matrix_entry[S](n, p, c, k, j) =
            matrix_entry[S](m, n, a, i, k) * matrix_entry[S](n, p, c, k, j) + matrix_entry[S](m, n, b, i, k) * matrix_entry[S](n, p, c, k, j)
        f(k) = matrix_entry[S](m, n, a, i, k) * matrix_entry[S](n, p, c, k, j)
        g(k) = matrix_entry[S](m, n, b, i, k) * matrix_entry[S](n, p, c, k, j)
        fin_pointwise_add[S](n, f, g, k) = f(k) + g(k)
        h(k) = fin_pointwise_add[S](n, f, g, k)
    }
    function_extensionality(h, fin_pointwise_add[S](n, f, g))
    fin_sum_add[S](n, f, g)
    matrix_entry_mul[S](m, n, p, a + b, c, i, j)
    matrix_entry_mul[S](m, n, p, a, c, i, j)
    matrix_entry_mul[S](m, n, p, b, c, i, j)
}

/// Matrix multiplication distributes over addition in the left factor.
theorem matrix_mul_add_left[S: Semiring](
    m: Nat, n: Nat, p: Nat, a: Matrix[S, m, n], b: Matrix[S, m, n], c: Matrix[S, n, p]
) {
    matrix_mul[S](m, n, p, a + b, c) = matrix_mul[S](m, n, p, a, c) + matrix_mul[S](m, n, p, b, c)
} by {
    let lhs = matrix_mul[S](m, n, p, a + b, c)
    let ac = matrix_mul[S](m, n, p, a, c)
    let bc = matrix_mul[S](m, n, p, b, c)
    let rhs = ac + bc
    forall(i: Fin[m], j: Fin[p]) {
        matrix_entry_mul_add_left[S](m, n, p, a, b, c, i, j)
        matrix_entry_add[S](m, p, ac, bc, i, j)
        matrix_entry[S](m, p, lhs, i, j) = matrix_entry[S](m, p, ac, i, j) + matrix_entry[S](m, p, bc, i, j)
        matrix_entry[S](m, p, rhs, i, j) = matrix_entry[S](m, p, ac, i, j) + matrix_entry[S](m, p, bc, i, j)
        matrix_entry[S](m, p, lhs, i, j) = matrix_entry[S](m, p, rhs, i, j)
    }
    matrix_ext[S](m, p, lhs, rhs)
    lhs = rhs
}

/// The reversed summand appearing in the transpose of a product.
define matrix_transpose_mul_rev_term[R: CommRing](m: Nat, n: Nat, p: Nat, a: Matrix[R, m, n], b: Matrix[R, n, p], i: Fin[m], j: Fin[p], k: Fin[n]) -> R {
    matrix_mul_term[R](p, n, m, matrix_transpose[R](n, p, b), matrix_transpose[R](m, n, a), j, i, k)
}

/// The original summand corresponding to a transpose-product summand.
define matrix_transpose_mul_orig_term[R: CommRing](m: Nat, n: Nat, p: Nat, a: Matrix[R, m, n], b: Matrix[R, n, p], i: Fin[m], j: Fin[p], k: Fin[n]) -> R {
    matrix_mul_term[R](m, n, p, a, b, i, j, k)
}

/// Over a commutative ring, the summands for the transpose of a product agree with the reversed product summands.
theorem matrix_transpose_mul_term_eq[R: CommRing](m: Nat, n: Nat, p: Nat, a: Matrix[R, m, n], b: Matrix[R, n, p], i: Fin[m], j: Fin[p], k: Fin[n]) {
    matrix_transpose_mul_rev_term[R](m, n, p, a, b, i, j, k) = matrix_transpose_mul_orig_term[R](m, n, p, a, b, i, j, k)
} by {
    matrix_transpose_mul_rev_term[R](m, n, p, a, b, i, j, k) =
        matrix_mul_term[R](p, n, m, matrix_transpose[R](n, p, b), matrix_transpose[R](m, n, a), j, i, k)
    matrix_transpose_mul_orig_term[R](m, n, p, a, b, i, j, k) = matrix_mul_term[R](m, n, p, a, b, i, j, k)
    matrix_mul_term[R](p, n, m, matrix_transpose[R](n, p, b), matrix_transpose[R](m, n, a), j, i, k) =
        matrix_entry[R](p, n, matrix_transpose[R](n, p, b), j, k) * matrix_entry[R](n, m, matrix_transpose[R](m, n, a), k, i)
    matrix_entry_transpose[R](n, p, b, k, j)
    matrix_entry_transpose[R](m, n, a, i, k)
    matrix_entry[R](p, n, matrix_transpose[R](n, p, b), j, k) = matrix_entry[R](n, p, b, k, j)
    matrix_entry[R](n, m, matrix_transpose[R](m, n, a), k, i) = matrix_entry[R](m, n, a, i, k)
    matrix_mul_term[R](p, n, m, matrix_transpose[R](n, p, b), matrix_transpose[R](m, n, a), j, i, k) =
        matrix_entry[R](n, p, b, k, j) * matrix_entry[R](m, n, a, i, k)
    matrix_mul_term[R](m, n, p, a, b, i, j, k) =
        matrix_entry[R](m, n, a, i, k) * matrix_entry[R](n, p, b, k, j)
    matrix_entry[R](n, p, b, k, j) * matrix_entry[R](m, n, a, i, k) =
        matrix_entry[R](m, n, a, i, k) * matrix_entry[R](n, p, b, k, j)
}

/// Over a commutative ring, transposition reverses matrix multiplication at each entry.
theorem matrix_entry_transpose_mul[R: CommRing](m: Nat, n: Nat, p: Nat, a: Matrix[R, m, n], b: Matrix[R, n, p], i: Fin[m], j: Fin[p]) {
    matrix_entry[R](p, m, matrix_transpose[R](m, p, matrix_mul[R](m, n, p, a, b)), j, i) =
        matrix_entry[R](p, m, matrix_mul[R](p, n, m, matrix_transpose[R](n, p, b), matrix_transpose[R](m, n, a)), j, i)
} by {
    let lhs = matrix_transpose[R](m, p, matrix_mul[R](m, n, p, a, b))
    let rhs = matrix_mul[R](p, n, m, matrix_transpose[R](n, p, b), matrix_transpose[R](m, n, a))
    matrix_entry_transpose[R](m, p, matrix_mul[R](m, n, p, a, b), i, j)
    matrix_entry_mul[R](m, n, p, a, b, i, j)
    matrix_entry_mul[R](p, n, m, matrix_transpose[R](n, p, b), matrix_transpose[R](m, n, a), j, i)
    forall(k: Fin[n]) {
        matrix_transpose_mul_term_eq[R](m, n, p, a, b, i, j, k)
        matrix_transpose_mul_rev_term[R](m, n, p, a, b, i, j, k) =
            matrix_mul_term[R](p, n, m, matrix_transpose[R](n, p, b), matrix_transpose[R](m, n, a), j, i, k)
        matrix_transpose_mul_orig_term[R](m, n, p, a, b, i, j, k) = matrix_mul_term[R](m, n, p, a, b, i, j, k)
        matrix_mul_term[R](p, n, m, matrix_transpose[R](n, p, b), matrix_transpose[R](m, n, a), j, i, k) =
            matrix_mul_term[R](m, n, p, a, b, i, j, k)
    }
    fin_sum_pointwise_eq[R](n,
        matrix_mul_term[R](p, n, m, matrix_transpose[R](n, p, b), matrix_transpose[R](m, n, a), j, i),
        matrix_mul_term[R](m, n, p, a, b, i, j))
    matrix_entry[R](p, m, lhs, j, i) = fin_sum[R](n, matrix_mul_term[R](m, n, p, a, b, i, j))
    matrix_entry[R](p, m, rhs, j, i) = fin_sum[R](n, matrix_mul_term[R](p, n, m, matrix_transpose[R](n, p, b), matrix_transpose[R](m, n, a), j, i))
    fin_sum[R](n, matrix_mul_term[R](p, n, m, matrix_transpose[R](n, p, b), matrix_transpose[R](m, n, a), j, i)) =
        fin_sum[R](n, matrix_mul_term[R](m, n, p, a, b, i, j))
    matrix_entry[R](p, m, lhs, j, i) = matrix_entry[R](p, m, rhs, j, i)
}

/// Over a commutative ring, transposition reverses matrix multiplication.
theorem matrix_transpose_mul[R: CommRing](m: Nat, n: Nat, p: Nat, a: Matrix[R, m, n], b: Matrix[R, n, p]) {
    matrix_transpose[R](m, p, matrix_mul[R](m, n, p, a, b)) =
        matrix_mul[R](p, n, m, matrix_transpose[R](n, p, b), matrix_transpose[R](m, n, a))
} by {
    let lhs = matrix_transpose[R](m, p, matrix_mul[R](m, n, p, a, b))
    let rhs = matrix_mul[R](p, n, m, matrix_transpose[R](n, p, b), matrix_transpose[R](m, n, a))
    forall(j: Fin[p], i: Fin[m]) {
        matrix_entry_transpose_mul[R](m, n, p, a, b, i, j)
        matrix_entry[R](p, m, lhs, j, i) = matrix_entry[R](p, m, rhs, j, i)
    }
    matrix_ext[R](p, m, lhs, rhs)
    lhs = rhs
}

/// Over a commutative ring, transposition reverses square matrix multiplication.
theorem matrix_transpose_square_mul[R: CommRing](n: Nat, a: Matrix[R, n, n], b: Matrix[R, n, n]) {
    matrix_transpose[R](n, n, square_matrix_mul[R](n, a, b)) =
        square_matrix_mul[R](n, matrix_transpose[R](n, n, b), matrix_transpose[R](n, n, a))
} by {
    matrix_transpose_mul[R](n, n, n, a, b)
}

/// Multiplying on the right by the identity square matrix changes nothing.
theorem square_matrix_mul_one_right[S: Semiring](n: Nat, a: Matrix[S, n, n]) {
    square_matrix_mul[S](n, a, matrix_one[S](n)) = a
} by {
    matrix_mul_one_right[S](n, n, a)
}

/// Multiplying on the left by the identity square matrix changes nothing.
theorem square_matrix_mul_one_left[S: Semiring](n: Nat, a: Matrix[S, n, n]) {
    square_matrix_mul[S](n, matrix_one[S](n), a) = a
} by {
    matrix_mul_one_left[S](n, n, a)
}

/// Square matrix multiplication distributes over addition in the right factor.
theorem square_matrix_mul_add_right[S: Semiring](n: Nat, a: Matrix[S, n, n], b: Matrix[S, n, n], c: Matrix[S, n, n]) {
    square_matrix_mul[S](n, a, b + c) = square_matrix_mul[S](n, a, b) + square_matrix_mul[S](n, a, c)
} by {
    matrix_mul_add_right[S](n, n, n, a, b, c)
}

/// Square matrix multiplication distributes over addition in the left factor.
theorem square_matrix_mul_add_left[S: Semiring](n: Nat, a: Matrix[S, n, n], b: Matrix[S, n, n], c: Matrix[S, n, n]) {
    square_matrix_mul[S](n, a + b, c) = square_matrix_mul[S](n, a, c) + square_matrix_mul[S](n, b, c)
} by {
    matrix_mul_add_left[S](n, n, n, a, b, c)
}

/// Pointwise matrix addition is associative.
theorem matrix_add_associative[A: AddSemigroup](m: Nat, n: Nat, a: Matrix[A, m, n], b: Matrix[A, m, n], c: Matrix[A, m, n]) {
    a + (b + c) = (a + b) + c
} by {
    let lhs = a + (b + c)
    let rhs = (a + b) + c
    forall(i: Fin[m], j: Fin[n]) {
        matrix_entry_add[A](m, n, a, b + c, i, j)
        matrix_entry_add[A](m, n, b, c, i, j)
        matrix_entry_add[A](m, n, a + b, c, i, j)
        matrix_entry_add[A](m, n, a, b, i, j)
        matrix_entry[A](m, n, lhs, i, j) = matrix_entry[A](m, n, a, i, j) + (matrix_entry[A](m, n, b, i, j) + matrix_entry[A](m, n, c, i, j))
        matrix_entry[A](m, n, rhs, i, j) = (matrix_entry[A](m, n, a, i, j) + matrix_entry[A](m, n, b, i, j)) + matrix_entry[A](m, n, c, i, j)
        matrix_entry[A](m, n, a, i, j) + (matrix_entry[A](m, n, b, i, j) + matrix_entry[A](m, n, c, i, j)) =
            (matrix_entry[A](m, n, a, i, j) + matrix_entry[A](m, n, b, i, j)) + matrix_entry[A](m, n, c, i, j)
        matrix_entry[A](m, n, lhs, i, j) = matrix_entry[A](m, n, rhs, i, j)
    }
    matrix_ext[A](m, n, lhs, rhs)
    lhs = rhs
}

/// Pointwise matrix addition is commutative.
theorem matrix_add_commutative[A: AddCommSemigroup](m: Nat, n: Nat, a: Matrix[A, m, n], b: Matrix[A, m, n]) {
    a + b = b + a
} by {
    let lhs = a + b
    let rhs = b + a
    forall(i: Fin[m], j: Fin[n]) {
        matrix_entry_add[A](m, n, a, b, i, j)
        matrix_entry_add[A](m, n, b, a, i, j)
        matrix_entry[A](m, n, a, i, j) + matrix_entry[A](m, n, b, i, j) = matrix_entry[A](m, n, b, i, j) + matrix_entry[A](m, n, a, i, j)
        matrix_entry[A](m, n, lhs, i, j) = matrix_entry[A](m, n, rhs, i, j)
    }
    matrix_ext[A](m, n, lhs, rhs)
    lhs = rhs
}

/// Adding the zero matrix on the right changes nothing.
theorem matrix_add_identity_right[A: AddMonoid](m: Nat, n: Nat, a: Matrix[A, m, n]) {
    a + matrix_zero[A](m, n) = a
} by {
    let z = matrix_zero[A](m, n)
    let lhs = a + z
    forall(i: Fin[m], j: Fin[n]) {
        matrix_entry_add[A](m, n, a, z, i, j)
        matrix_entry_zero[A](m, n, i, j)
        matrix_entry[A](m, n, a, i, j) + A.0 = matrix_entry[A](m, n, a, i, j)
        matrix_entry[A](m, n, lhs, i, j) = matrix_entry[A](m, n, a, i, j)
    }
    matrix_ext[A](m, n, lhs, a)
    lhs = a
}

/// Adding the zero matrix on the left changes nothing.
theorem matrix_add_identity_left[A: AddMonoid](m: Nat, n: Nat, a: Matrix[A, m, n]) {
    matrix_zero[A](m, n) + a = a
} by {
    let z = matrix_zero[A](m, n)
    let lhs = z + a
    forall(i: Fin[m], j: Fin[n]) {
        matrix_entry_add[A](m, n, z, a, i, j)
        matrix_entry_zero[A](m, n, i, j)
        A.0 + matrix_entry[A](m, n, a, i, j) = matrix_entry[A](m, n, a, i, j)
        matrix_entry[A](m, n, lhs, i, j) = matrix_entry[A](m, n, a, i, j)
    }
    matrix_ext[A](m, n, lhs, a)
    lhs = a
}

/// Pointwise matrix negation is a right additive inverse.
theorem matrix_inverse_right[A: AddGroup](m: Nat, n: Nat, a: Matrix[A, m, n]) {
    a + -a = matrix_zero[A](m, n)
} by {
    let z = matrix_zero[A](m, n)
    let lhs = a + -a
    forall(i: Fin[m], j: Fin[n]) {
        matrix_entry_add[A](m, n, a, -a, i, j)
        matrix_entry_neg[A](m, n, a, i, j)
        matrix_entry_zero[A](m, n, i, j)
        matrix_entry[A](m, n, a, i, j) + -matrix_entry[A](m, n, a, i, j) = A.0
        matrix_entry[A](m, n, lhs, i, j) = matrix_entry[A](m, n, z, i, j)
    }
    matrix_ext[A](m, n, lhs, z)
    lhs = z
}

/// Pointwise matrix subtraction is addition with pointwise negation.
theorem matrix_sub_eq_add_neg[A: AddGroup](m: Nat, n: Nat, a: Matrix[A, m, n], b: Matrix[A, m, n]) {
    matrix_sub[A](m, n, a, b) = a + -b
} by {
    let lhs = matrix_sub[A](m, n, a, b)
    let rhs = a + -b
    forall(i: Fin[m], j: Fin[n]) {
        matrix_entry_sub[A](m, n, a, b, i, j)
        matrix_entry_add[A](m, n, a, -b, i, j)
        matrix_entry_neg[A](m, n, b, i, j)
        matrix_entry[A](m, n, a, i, j) - matrix_entry[A](m, n, b, i, j) =
            matrix_entry[A](m, n, a, i, j) + -matrix_entry[A](m, n, b, i, j)
        matrix_entry[A](m, n, lhs, i, j) = matrix_entry[A](m, n, rhs, i, j)
    }
    matrix_ext[A](m, n, lhs, rhs)
    lhs = rhs
}

/// Transposing a matrix twice gives the original matrix.
theorem matrix_transpose_transpose[A](m: Nat, n: Nat, a: Matrix[A, m, n]) {
    matrix_transpose[A](n, m, matrix_transpose[A](m, n, a)) = a
} by {
    let lhs = matrix_transpose[A](n, m, matrix_transpose[A](m, n, a))
    forall(i: Fin[m], j: Fin[n]) {
        matrix_entry[A](m, n, lhs, i, j) = matrix_transpose_entry[A](n, m, matrix_transpose[A](m, n, a), i, j)
        matrix_transpose_entry[A](n, m, matrix_transpose[A](m, n, a), i, j) =
            matrix_entry[A](n, m, matrix_transpose[A](m, n, a), j, i)
        matrix_entry[A](n, m, matrix_transpose[A](m, n, a), j, i) = matrix_transpose_entry[A](m, n, a, j, i)
        matrix_transpose_entry[A](m, n, a, j, i) = matrix_entry[A](m, n, a, i, j)
        matrix_entry[A](m, n, lhs, i, j) = matrix_entry[A](m, n, a, i, j)
    }
    matrix_ext[A](m, n, lhs, a)
    lhs = a
}

/// Rows of the transpose are columns of the original matrix at each coordinate.
theorem matrix_row_transpose[A](m: Nat, n: Nat, a: Matrix[A, m, n], j: Fin[n], i: Fin[m]) {
    matrix_row[A](n, m, matrix_transpose[A](m, n, a), j, i) = matrix_col[A](m, n, a, j, i)
} by {
    matrix_row_entry[A](n, m, matrix_transpose[A](m, n, a), j, i)
    matrix_entry[A](n, m, matrix_transpose[A](m, n, a), j, i) = matrix_transpose_entry[A](m, n, a, j, i)
    matrix_transpose_entry[A](m, n, a, j, i) = matrix_entry[A](m, n, a, i, j)
    matrix_col_entry[A](m, n, a, j, i)
}

/// Columns of the transpose are rows of the original matrix at each coordinate.
theorem matrix_col_transpose[A](m: Nat, n: Nat, a: Matrix[A, m, n], i: Fin[m], j: Fin[n]) {
    matrix_col[A](n, m, matrix_transpose[A](m, n, a), i, j) = matrix_row[A](m, n, a, i, j)
} by {
    matrix_col_entry[A](n, m, matrix_transpose[A](m, n, a), i, j)
    matrix_entry[A](n, m, matrix_transpose[A](m, n, a), j, i) = matrix_transpose_entry[A](m, n, a, j, i)
    matrix_transpose_entry[A](m, n, a, j, i) = matrix_entry[A](m, n, a, i, j)
    matrix_row_entry[A](m, n, a, i, j)
}

/// A row of a transpose is the corresponding column of the original matrix.
theorem matrix_row_transpose_vector[A](m: Nat, n: Nat, a: Matrix[A, m, n], j: Fin[n]) {
    matrix_row[A](n, m, matrix_transpose[A](m, n, a), j) = matrix_col[A](m, n, a, j)
} by {
    let lhs = matrix_row[A](n, m, matrix_transpose[A](m, n, a), j)
    let rhs = matrix_col[A](m, n, a, j)
    forall(i: Fin[m]) {
        lhs(i) = matrix_entry[A](n, m, matrix_transpose[A](m, n, a), j, i)
        matrix_entry_transpose[A](m, n, a, i, j)
        rhs(i) = matrix_entry[A](m, n, a, i, j)
        lhs(i) = rhs(i)
    }
    function_extensionality(lhs, rhs)
}

/// A column of a transpose is the corresponding row of the original matrix.
theorem matrix_col_transpose_vector[A](m: Nat, n: Nat, a: Matrix[A, m, n], i: Fin[m]) {
    matrix_col[A](n, m, matrix_transpose[A](m, n, a), i) = matrix_row[A](m, n, a, i)
} by {
    let lhs = matrix_col[A](n, m, matrix_transpose[A](m, n, a), i)
    let rhs = matrix_row[A](m, n, a, i)
    forall(j: Fin[n]) {
        lhs(j) = matrix_entry[A](n, m, matrix_transpose[A](m, n, a), j, i)
        matrix_entry_transpose[A](m, n, a, i, j)
        rhs(j) = matrix_entry[A](m, n, a, i, j)
        lhs(j) = rhs(j)
    }
    function_extensionality(lhs, rhs)
}

/// Rows of the zero matrix are zero coordinate vectors.
theorem matrix_row_zero[A: Zero](m: Nat, n: Nat, i: Fin[m]) {
    matrix_row[A](m, n, matrix_zero[A](m, n), i) = fin_vector_zero[A](n)
} by {
    forall(j: Fin[n]) {
        matrix_row_entry[A](m, n, matrix_zero[A](m, n), i, j)
        matrix_entry_zero[A](m, n, i, j)
        matrix_row[A](m, n, matrix_zero[A](m, n), i, j) = A.0
        fin_vector_zero[A](n, j) = A.0
        matrix_row[A](m, n, matrix_zero[A](m, n), i, j) = fin_vector_zero[A](n, j)
    }
    function_extensionality(matrix_row[A](m, n, matrix_zero[A](m, n), i), fin_vector_zero[A](n))
}

/// Columns of the zero matrix are zero coordinate vectors.
theorem matrix_col_zero[A: Zero](m: Nat, n: Nat, j: Fin[n]) {
    matrix_col[A](m, n, matrix_zero[A](m, n), j) = fin_vector_zero[A](m)
} by {
    forall(i: Fin[m]) {
        matrix_col_entry[A](m, n, matrix_zero[A](m, n), j, i)
        matrix_entry_zero[A](m, n, i, j)
        matrix_col[A](m, n, matrix_zero[A](m, n), j, i) = A.0
        fin_vector_zero[A](m, i) = A.0
        matrix_col[A](m, n, matrix_zero[A](m, n), j, i) = fin_vector_zero[A](m, i)
    }
    function_extensionality(matrix_col[A](m, n, matrix_zero[A](m, n), j), fin_vector_zero[A](m))
}

/// Taking a row commutes with pointwise matrix addition.
theorem matrix_row_add[A: Add](m: Nat, n: Nat, a: Matrix[A, m, n], b: Matrix[A, m, n], i: Fin[m]) {
    matrix_row[A](m, n, a + b, i) = fin_vector_add[A](n, matrix_row[A](m, n, a, i), matrix_row[A](m, n, b, i))
} by {
    forall(j: Fin[n]) {
        matrix_row_entry[A](m, n, a + b, i, j)
        matrix_entry_add[A](m, n, a, b, i, j)
        matrix_row_entry[A](m, n, a, i, j)
        matrix_row_entry[A](m, n, b, i, j)
        matrix_row[A](m, n, a + b, i, j) = matrix_entry[A](m, n, a, i, j) + matrix_entry[A](m, n, b, i, j)
        fin_vector_add[A](n, matrix_row[A](m, n, a, i), matrix_row[A](m, n, b, i), j) =
            matrix_row[A](m, n, a, i, j) + matrix_row[A](m, n, b, i, j)
        matrix_row[A](m, n, a + b, i, j) = fin_vector_add[A](n, matrix_row[A](m, n, a, i), matrix_row[A](m, n, b, i), j)
    }
    function_extensionality(matrix_row[A](m, n, a + b, i), fin_vector_add[A](n, matrix_row[A](m, n, a, i), matrix_row[A](m, n, b, i)))
}

/// Taking a column commutes with pointwise matrix addition.
theorem matrix_col_add[A: Add](m: Nat, n: Nat, a: Matrix[A, m, n], b: Matrix[A, m, n], j: Fin[n]) {
    matrix_col[A](m, n, a + b, j) = fin_vector_add[A](m, matrix_col[A](m, n, a, j), matrix_col[A](m, n, b, j))
} by {
    forall(i: Fin[m]) {
        matrix_col_entry[A](m, n, a + b, j, i)
        matrix_entry_add[A](m, n, a, b, i, j)
        matrix_col_entry[A](m, n, a, j, i)
        matrix_col_entry[A](m, n, b, j, i)
        matrix_col[A](m, n, a + b, j, i) = matrix_entry[A](m, n, a, i, j) + matrix_entry[A](m, n, b, i, j)
        fin_vector_add[A](m, matrix_col[A](m, n, a, j), matrix_col[A](m, n, b, j), i) =
            matrix_col[A](m, n, a, j, i) + matrix_col[A](m, n, b, j, i)
        matrix_col[A](m, n, a + b, j, i) = fin_vector_add[A](m, matrix_col[A](m, n, a, j), matrix_col[A](m, n, b, j), i)
    }
    function_extensionality(matrix_col[A](m, n, a + b, j), fin_vector_add[A](m, matrix_col[A](m, n, a, j), matrix_col[A](m, n, b, j)))
}

/// Taking a row commutes with pointwise matrix negation.
theorem matrix_row_neg[A: Neg](m: Nat, n: Nat, a: Matrix[A, m, n], i: Fin[m]) {
    matrix_row[A](m, n, -a, i) = fin_vector_neg[A](n, matrix_row[A](m, n, a, i))
} by {
    let neg_a = -a
    forall(j: Fin[n]) {
        matrix_row_entry[A](m, n, neg_a, i, j)
        matrix_entry_neg[A](m, n, a, i, j)
        matrix_row_entry[A](m, n, a, i, j)
        fin_vector_neg[A](n, matrix_row[A](m, n, a, i), j) = -matrix_row[A](m, n, a, i, j)
        matrix_row[A](m, n, neg_a, i, j) = fin_vector_neg[A](n, matrix_row[A](m, n, a, i), j)
    }
    function_extensionality(matrix_row[A](m, n, neg_a, i), fin_vector_neg[A](n, matrix_row[A](m, n, a, i)))
}

/// Taking a column commutes with pointwise matrix negation.
theorem matrix_col_neg[A: Neg](m: Nat, n: Nat, a: Matrix[A, m, n], j: Fin[n]) {
    matrix_col[A](m, n, -a, j) = fin_vector_neg[A](m, matrix_col[A](m, n, a, j))
} by {
    let neg_a = -a
    forall(i: Fin[m]) {
        matrix_col_entry[A](m, n, neg_a, j, i)
        matrix_entry_neg[A](m, n, a, i, j)
        matrix_col_entry[A](m, n, a, j, i)
        fin_vector_neg[A](m, matrix_col[A](m, n, a, j), i) = -matrix_col[A](m, n, a, j, i)
        matrix_col[A](m, n, neg_a, j, i) = fin_vector_neg[A](m, matrix_col[A](m, n, a, j), i)
    }
    function_extensionality(matrix_col[A](m, n, neg_a, j), fin_vector_neg[A](m, matrix_col[A](m, n, a, j)))
}

/// Taking a row commutes with pointwise matrix subtraction.
theorem matrix_row_sub[A: AddGroup](m: Nat, n: Nat, a: Matrix[A, m, n], b: Matrix[A, m, n], i: Fin[m]) {
    matrix_row[A](m, n, matrix_sub[A](m, n, a, b), i) = fin_vector_sub[A](n, matrix_row[A](m, n, a, i), matrix_row[A](m, n, b, i))
} by {
    forall(j: Fin[n]) {
        matrix_row_entry[A](m, n, matrix_sub[A](m, n, a, b), i, j)
        matrix_entry_sub[A](m, n, a, b, i, j)
        matrix_row_entry[A](m, n, a, i, j)
        matrix_row_entry[A](m, n, b, i, j)
        fin_vector_sub[A](n, matrix_row[A](m, n, a, i), matrix_row[A](m, n, b, i), j) =
            matrix_row[A](m, n, a, i, j) - matrix_row[A](m, n, b, i, j)
        matrix_row[A](m, n, matrix_sub[A](m, n, a, b), i, j) = fin_vector_sub[A](n, matrix_row[A](m, n, a, i), matrix_row[A](m, n, b, i), j)
    }
    function_extensionality(matrix_row[A](m, n, matrix_sub[A](m, n, a, b), i), fin_vector_sub[A](n, matrix_row[A](m, n, a, i), matrix_row[A](m, n, b, i)))
}

/// Taking a column commutes with pointwise matrix subtraction.
theorem matrix_col_sub[A: AddGroup](m: Nat, n: Nat, a: Matrix[A, m, n], b: Matrix[A, m, n], j: Fin[n]) {
    matrix_col[A](m, n, matrix_sub[A](m, n, a, b), j) = fin_vector_sub[A](m, matrix_col[A](m, n, a, j), matrix_col[A](m, n, b, j))
} by {
    forall(i: Fin[m]) {
        matrix_col_entry[A](m, n, matrix_sub[A](m, n, a, b), j, i)
        matrix_entry_sub[A](m, n, a, b, i, j)
        matrix_col_entry[A](m, n, a, j, i)
        matrix_col_entry[A](m, n, b, j, i)
        fin_vector_sub[A](m, matrix_col[A](m, n, a, j), matrix_col[A](m, n, b, j), i) =
            matrix_col[A](m, n, a, j, i) - matrix_col[A](m, n, b, j, i)
        matrix_col[A](m, n, matrix_sub[A](m, n, a, b), j, i) = fin_vector_sub[A](m, matrix_col[A](m, n, a, j), matrix_col[A](m, n, b, j), i)
    }
    function_extensionality(matrix_col[A](m, n, matrix_sub[A](m, n, a, b), j), fin_vector_sub[A](m, matrix_col[A](m, n, a, j), matrix_col[A](m, n, b, j)))
}

/// Taking a row commutes with entrywise scalar multiplication.
theorem matrix_row_smul[R: Ring, M: AddCommGroup](carrier: Module[R, M], m: Nat, n: Nat, r: R, a: Matrix[M, m, n], i: Fin[m]) {
    matrix_row[M](m, n, matrix_smul[R, M](carrier, m, n, r, a), i) =
        fin_vector_smul[R, M](carrier, n, r, matrix_row[M](m, n, a, i))
} by {
    forall(j: Fin[n]) {
        matrix_row_entry[M](m, n, matrix_smul[R, M](carrier, m, n, r, a), i, j)
        matrix_entry_smul[R, M](carrier, m, n, r, a, i, j)
        matrix_row_entry[M](m, n, a, i, j)
        fin_vector_smul[R, M](carrier, n, r, matrix_row[M](m, n, a, i), j) =
            carrier.smul(r, matrix_row[M](m, n, a, i, j))
        matrix_row[M](m, n, matrix_smul[R, M](carrier, m, n, r, a), i, j) =
            fin_vector_smul[R, M](carrier, n, r, matrix_row[M](m, n, a, i), j)
    }
    function_extensionality(matrix_row[M](m, n, matrix_smul[R, M](carrier, m, n, r, a), i), fin_vector_smul[R, M](carrier, n, r, matrix_row[M](m, n, a, i)))
}

/// Taking a column commutes with entrywise scalar multiplication.
theorem matrix_col_smul[R: Ring, M: AddCommGroup](carrier: Module[R, M], m: Nat, n: Nat, r: R, a: Matrix[M, m, n], j: Fin[n]) {
    matrix_col[M](m, n, matrix_smul[R, M](carrier, m, n, r, a), j) =
        fin_vector_smul[R, M](carrier, m, r, matrix_col[M](m, n, a, j))
} by {
    forall(i: Fin[m]) {
        matrix_col_entry[M](m, n, matrix_smul[R, M](carrier, m, n, r, a), j, i)
        matrix_entry_smul[R, M](carrier, m, n, r, a, i, j)
        matrix_col_entry[M](m, n, a, j, i)
        fin_vector_smul[R, M](carrier, m, r, matrix_col[M](m, n, a, j), i) =
            carrier.smul(r, matrix_col[M](m, n, a, j, i))
        matrix_col[M](m, n, matrix_smul[R, M](carrier, m, n, r, a), j, i) =
            fin_vector_smul[R, M](carrier, m, r, matrix_col[M](m, n, a, j), i)
    }
    function_extensionality(matrix_col[M](m, n, matrix_smul[R, M](carrier, m, n, r, a), j), fin_vector_smul[R, M](carrier, m, r, matrix_col[M](m, n, a, j)))
}

/// The transpose of the zero matrix is the zero matrix.
theorem matrix_transpose_zero[A: Zero](m: Nat, n: Nat) {
    matrix_transpose[A](m, n, matrix_zero[A](m, n)) = matrix_zero[A](n, m)
} by {
    let lhs = matrix_transpose[A](m, n, matrix_zero[A](m, n))
    let rhs = matrix_zero[A](n, m)
    forall(j: Fin[n], i: Fin[m]) {
        matrix_entry_transpose[A](m, n, matrix_zero[A](m, n), i, j)
        matrix_entry_zero[A](m, n, i, j)
        matrix_entry_zero[A](n, m, j, i)
        matrix_entry[A](n, m, lhs, j, i) = A.0
        matrix_entry[A](n, m, rhs, j, i) = A.0
        matrix_entry[A](n, m, lhs, j, i) = matrix_entry[A](n, m, rhs, j, i)
    }
    matrix_ext[A](n, m, lhs, rhs)
    lhs = rhs
}

/// Transposition distributes over matrix addition.
theorem matrix_transpose_add[A: Add](m: Nat, n: Nat, a: Matrix[A, m, n], b: Matrix[A, m, n]) {
    matrix_transpose[A](m, n, a + b) = matrix_transpose[A](m, n, a) + matrix_transpose[A](m, n, b)
} by {
    let lhs = matrix_transpose[A](m, n, a + b)
    let at = matrix_transpose[A](m, n, a)
    let bt = matrix_transpose[A](m, n, b)
    let rhs = at + bt
    forall(j: Fin[n], i: Fin[m]) {
        matrix_entry_transpose[A](m, n, a + b, i, j)
        matrix_entry_add[A](m, n, a, b, i, j)
        matrix_entry_transpose[A](m, n, a, i, j)
        matrix_entry_transpose[A](m, n, b, i, j)
        matrix_entry_add[A](n, m, at, bt, j, i)
        matrix_entry[A](n, m, lhs, j, i) = matrix_entry[A](m, n, a, i, j) + matrix_entry[A](m, n, b, i, j)
        matrix_entry[A](n, m, rhs, j, i) = matrix_entry[A](m, n, a, i, j) + matrix_entry[A](m, n, b, i, j)
        matrix_entry[A](n, m, lhs, j, i) = matrix_entry[A](n, m, rhs, j, i)
    }
    matrix_ext[A](n, m, lhs, rhs)
    lhs = rhs
}

/// Transposition distributes over matrix negation.
theorem matrix_transpose_neg[A: Neg](m: Nat, n: Nat, a: Matrix[A, m, n]) {
    matrix_transpose[A](m, n, -a) = -matrix_transpose[A](m, n, a)
} by {
    let lhs = matrix_transpose[A](m, n, -a)
    let at = matrix_transpose[A](m, n, a)
    let rhs = -at
    forall(j: Fin[n], i: Fin[m]) {
        matrix_entry_transpose[A](m, n, -a, i, j)
        matrix_entry_neg[A](m, n, a, i, j)
        matrix_entry_transpose[A](m, n, a, i, j)
        matrix_entry_neg[A](n, m, at, j, i)
        matrix_entry[A](n, m, lhs, j, i) = -matrix_entry[A](m, n, a, i, j)
        matrix_entry[A](n, m, rhs, j, i) = -matrix_entry[A](m, n, a, i, j)
        matrix_entry[A](n, m, lhs, j, i) = matrix_entry[A](n, m, rhs, j, i)
    }
    matrix_ext[A](n, m, lhs, rhs)
    lhs = rhs
}

/// Transposition distributes over matrix subtraction.
theorem matrix_transpose_sub[A: AddGroup](m: Nat, n: Nat, a: Matrix[A, m, n], b: Matrix[A, m, n]) {
    matrix_transpose[A](m, n, matrix_sub[A](m, n, a, b)) = matrix_sub[A](n, m, matrix_transpose[A](m, n, a), matrix_transpose[A](m, n, b))
} by {
    let lhs = matrix_transpose[A](m, n, matrix_sub[A](m, n, a, b))
    let at = matrix_transpose[A](m, n, a)
    let bt = matrix_transpose[A](m, n, b)
    let rhs = matrix_sub[A](n, m, at, bt)
    forall(j: Fin[n], i: Fin[m]) {
        matrix_entry_transpose[A](m, n, matrix_sub[A](m, n, a, b), i, j)
        matrix_entry_sub[A](m, n, a, b, i, j)
        matrix_entry_transpose[A](m, n, a, i, j)
        matrix_entry_transpose[A](m, n, b, i, j)
        matrix_entry_sub[A](n, m, at, bt, j, i)
        matrix_entry[A](n, m, lhs, j, i) = matrix_entry[A](m, n, a, i, j) - matrix_entry[A](m, n, b, i, j)
        matrix_entry[A](n, m, rhs, j, i) = matrix_entry[A](m, n, a, i, j) - matrix_entry[A](m, n, b, i, j)
        matrix_entry[A](n, m, lhs, j, i) = matrix_entry[A](n, m, rhs, j, i)
    }
    matrix_ext[A](n, m, lhs, rhs)
    lhs = rhs
}

/// The identity scalar fixes every matrix entry.
theorem matrix_smul_one_entry[R: Ring, M: AddCommGroup](carrier: Module[R, M], m: Nat, n: Nat, a: Matrix[M, m, n], i: Fin[m], j: Fin[n]) {
    matrix_entry[M](m, n, matrix_smul[R, M](carrier, m, n, R.1, a), i, j) = matrix_entry[M](m, n, a, i, j)
} by {
    matrix_entry_smul[R, M](carrier, m, n, R.1, a, i, j)
    module_smul_one(carrier, matrix_entry[M](m, n, a, i, j))
}

/// The zero scalar annihilates every matrix entry.
theorem matrix_smul_zero_left_entry[R: Ring, M: AddCommGroup](carrier: Module[R, M], m: Nat, n: Nat, a: Matrix[M, m, n], i: Fin[m], j: Fin[n]) {
    matrix_entry[M](m, n, matrix_smul[R, M](carrier, m, n, R.0, a), i, j) = M.0
} by {
    matrix_entry_smul[R, M](carrier, m, n, R.0, a, i, j)
    module_smul_zero_left(carrier, matrix_entry[M](m, n, a, i, j))
}

/// The zero scalar annihilates every matrix.
theorem matrix_smul_zero_left[R: Ring, M: AddCommGroup](carrier: Module[R, M], m: Nat, n: Nat, a: Matrix[M, m, n]) {
    matrix_smul[R, M](carrier, m, n, R.0, a) = matrix_zero[M](m, n)
} by {
    let lhs = matrix_smul[R, M](carrier, m, n, R.0, a)
    let rhs = matrix_zero[M](m, n)
    forall(i: Fin[m], j: Fin[n]) {
        matrix_smul_zero_left_entry[R, M](carrier, m, n, a, i, j)
        matrix_entry_zero[M](m, n, i, j)
        matrix_entry[M](m, n, lhs, i, j) = M.0
        matrix_entry[M](m, n, rhs, i, j) = M.0
        matrix_entry[M](m, n, lhs, i, j) = matrix_entry[M](m, n, rhs, i, j)
    }
    matrix_ext[M](m, n, lhs, rhs)
    lhs = rhs
}

/// Every scalar annihilates the zero matrix.
theorem matrix_smul_zero_right[R: Ring, M: AddCommGroup](carrier: Module[R, M], m: Nat, n: Nat, r: R) {
    matrix_smul[R, M](carrier, m, n, r, matrix_zero[M](m, n)) = matrix_zero[M](m, n)
} by {
    let z = matrix_zero[M](m, n)
    let lhs = matrix_smul[R, M](carrier, m, n, r, z)
    let rhs = matrix_zero[M](m, n)
    forall(i: Fin[m], j: Fin[n]) {
        matrix_entry_smul[R, M](carrier, m, n, r, z, i, j)
        matrix_entry_zero[M](m, n, i, j)
        module_smul_zero_right(carrier, r)
        matrix_entry[M](m, n, lhs, i, j) = carrier.smul(r, M.0)
        carrier.smul(r, M.0) = M.0
        matrix_entry[M](m, n, rhs, i, j) = M.0
        matrix_entry[M](m, n, lhs, i, j) = matrix_entry[M](m, n, rhs, i, j)
    }
    matrix_ext[M](m, n, lhs, rhs)
    lhs = rhs
}

/// Scalar multiplication distributes over scalar addition on matrices.
theorem matrix_smul_add_left[R: Ring, M: AddCommGroup](carrier: Module[R, M], m: Nat, n: Nat, r: R, s: R, a: Matrix[M, m, n]) {
    matrix_smul[R, M](carrier, m, n, r + s, a) = matrix_smul[R, M](carrier, m, n, r, a) + matrix_smul[R, M](carrier, m, n, s, a)
} by {
    let lhs = matrix_smul[R, M](carrier, m, n, r + s, a)
    let ra = matrix_smul[R, M](carrier, m, n, r, a)
    let sa = matrix_smul[R, M](carrier, m, n, s, a)
    let rhs = ra + sa
    forall(i: Fin[m], j: Fin[n]) {
        matrix_entry_smul[R, M](carrier, m, n, r + s, a, i, j)
        matrix_entry_smul[R, M](carrier, m, n, r, a, i, j)
        matrix_entry_smul[R, M](carrier, m, n, s, a, i, j)
        matrix_entry_add[M](m, n, ra, sa, i, j)
        module_smul_add_left(carrier, r, s, matrix_entry[M](m, n, a, i, j))
        matrix_entry[M](m, n, lhs, i, j) = carrier.smul(r, matrix_entry[M](m, n, a, i, j)) + carrier.smul(s, matrix_entry[M](m, n, a, i, j))
        matrix_entry[M](m, n, rhs, i, j) = carrier.smul(r, matrix_entry[M](m, n, a, i, j)) + carrier.smul(s, matrix_entry[M](m, n, a, i, j))
        matrix_entry[M](m, n, lhs, i, j) = matrix_entry[M](m, n, rhs, i, j)
    }
    matrix_ext[M](m, n, lhs, rhs)
    lhs = rhs
}

/// Scalar multiplication distributes over matrix addition.
theorem matrix_smul_add_right[R: Ring, M: AddCommGroup](carrier: Module[R, M], m: Nat, n: Nat, r: R, a: Matrix[M, m, n], b: Matrix[M, m, n]) {
    matrix_smul[R, M](carrier, m, n, r, a + b) = matrix_smul[R, M](carrier, m, n, r, a) + matrix_smul[R, M](carrier, m, n, r, b)
} by {
    let lhs = matrix_smul[R, M](carrier, m, n, r, a + b)
    let ra = matrix_smul[R, M](carrier, m, n, r, a)
    let rb = matrix_smul[R, M](carrier, m, n, r, b)
    let rhs = ra + rb
    forall(i: Fin[m], j: Fin[n]) {
        matrix_entry_smul[R, M](carrier, m, n, r, a + b, i, j)
        matrix_entry_add[M](m, n, a, b, i, j)
        matrix_entry_smul[R, M](carrier, m, n, r, a, i, j)
        matrix_entry_smul[R, M](carrier, m, n, r, b, i, j)
        matrix_entry_add[M](m, n, ra, rb, i, j)
        module_smul_add_right(carrier, r, matrix_entry[M](m, n, a, i, j), matrix_entry[M](m, n, b, i, j))
        matrix_entry[M](m, n, lhs, i, j) = carrier.smul(r, matrix_entry[M](m, n, a, i, j)) + carrier.smul(r, matrix_entry[M](m, n, b, i, j))
        matrix_entry[M](m, n, rhs, i, j) = carrier.smul(r, matrix_entry[M](m, n, a, i, j)) + carrier.smul(r, matrix_entry[M](m, n, b, i, j))
        matrix_entry[M](m, n, lhs, i, j) = matrix_entry[M](m, n, rhs, i, j)
    }
    matrix_ext[M](m, n, lhs, rhs)
    lhs = rhs
}

/// Scalar multiplication is compatible with ring multiplication on matrices.
theorem matrix_smul_assoc[R: Ring, M: AddCommGroup](carrier: Module[R, M], m: Nat, n: Nat, r: R, s: R, a: Matrix[M, m, n]) {
    matrix_smul[R, M](carrier, m, n, r * s, a) = matrix_smul[R, M](carrier, m, n, r, matrix_smul[R, M](carrier, m, n, s, a))
} by {
    let lhs = matrix_smul[R, M](carrier, m, n, r * s, a)
    let sa = matrix_smul[R, M](carrier, m, n, s, a)
    let rhs = matrix_smul[R, M](carrier, m, n, r, sa)
    forall(i: Fin[m], j: Fin[n]) {
        matrix_entry_smul[R, M](carrier, m, n, r * s, a, i, j)
        matrix_entry_smul[R, M](carrier, m, n, s, a, i, j)
        matrix_entry_smul[R, M](carrier, m, n, r, sa, i, j)
        module_smul_assoc(carrier, r, s, matrix_entry[M](m, n, a, i, j))
        matrix_entry[M](m, n, lhs, i, j) = carrier.smul(r, carrier.smul(s, matrix_entry[M](m, n, a, i, j)))
        matrix_entry[M](m, n, rhs, i, j) = carrier.smul(r, carrier.smul(s, matrix_entry[M](m, n, a, i, j)))
        matrix_entry[M](m, n, lhs, i, j) = matrix_entry[M](m, n, rhs, i, j)
    }
    matrix_ext[M](m, n, lhs, rhs)
    lhs = rhs
}

/// Negating the scalar negates a matrix scalar multiple.
theorem matrix_smul_neg_left[R: Ring, M: AddCommGroup](carrier: Module[R, M], m: Nat, n: Nat, r: R, a: Matrix[M, m, n]) {
    matrix_smul[R, M](carrier, m, n, -r, a) = -matrix_smul[R, M](carrier, m, n, r, a)
} by {
    let lhs = matrix_smul[R, M](carrier, m, n, -r, a)
    let ra = matrix_smul[R, M](carrier, m, n, r, a)
    let rhs = -ra
    forall(i: Fin[m], j: Fin[n]) {
        matrix_entry_smul[R, M](carrier, m, n, -r, a, i, j)
        matrix_entry_smul[R, M](carrier, m, n, r, a, i, j)
        matrix_entry_neg[M](m, n, ra, i, j)
        module_smul_neg_left(carrier, r, matrix_entry[M](m, n, a, i, j))
        matrix_entry[M](m, n, lhs, i, j) = -carrier.smul(r, matrix_entry[M](m, n, a, i, j))
        matrix_entry[M](m, n, rhs, i, j) = -carrier.smul(r, matrix_entry[M](m, n, a, i, j))
        matrix_entry[M](m, n, lhs, i, j) = matrix_entry[M](m, n, rhs, i, j)
    }
    matrix_ext[M](m, n, lhs, rhs)
    lhs = rhs
}

/// Scalar multiplication distributes over matrix negation.
theorem matrix_smul_neg_right[R: Ring, M: AddCommGroup](carrier: Module[R, M], m: Nat, n: Nat, r: R, a: Matrix[M, m, n]) {
    matrix_smul[R, M](carrier, m, n, r, -a) = -matrix_smul[R, M](carrier, m, n, r, a)
} by {
    let lhs = matrix_smul[R, M](carrier, m, n, r, -a)
    let ra = matrix_smul[R, M](carrier, m, n, r, a)
    let rhs = -ra
    forall(i: Fin[m], j: Fin[n]) {
        matrix_entry_smul[R, M](carrier, m, n, r, -a, i, j)
        matrix_entry_neg[M](m, n, a, i, j)
        matrix_entry_smul[R, M](carrier, m, n, r, a, i, j)
        matrix_entry_neg[M](m, n, ra, i, j)
        module_smul_neg_right(carrier, r, matrix_entry[M](m, n, a, i, j))
        matrix_entry[M](m, n, lhs, i, j) = -carrier.smul(r, matrix_entry[M](m, n, a, i, j))
        matrix_entry[M](m, n, rhs, i, j) = -carrier.smul(r, matrix_entry[M](m, n, a, i, j))
        matrix_entry[M](m, n, lhs, i, j) = matrix_entry[M](m, n, rhs, i, j)
    }
    matrix_ext[M](m, n, lhs, rhs)
    lhs = rhs
}

/// The zeroth power of a square matrix is the identity matrix.
theorem matrix_pow_zero[S: Semiring](n: Nat, a: Matrix[S, n, n]) {
    matrix_pow[S](n, a, Nat.0) = matrix_one[S](n)
}

/// The successor power is left multiplication by the base matrix.
theorem matrix_pow_suc[S: Semiring](n: Nat, a: Matrix[S, n, n], k: Nat) {
    matrix_pow[S](n, a, k.suc) = square_matrix_mul[S](n, a, matrix_pow[S](n, a, k))
}

/// The first power of a square matrix is its product with the identity matrix.
theorem matrix_pow_one_unfold[S: Semiring](n: Nat, a: Matrix[S, n, n]) {
    matrix_pow[S](n, a, Nat.1) = square_matrix_mul[S](n, a, matrix_one[S](n))
} by {
    matrix_pow_suc[S](n, a, Nat.0)
    matrix_pow_zero[S](n, a)
}

/// The first power of a square matrix is itself.
theorem matrix_pow_one[S: Semiring](n: Nat, a: Matrix[S, n, n]) {
    matrix_pow[S](n, a, Nat.1) = a
} by {
    matrix_pow_one_unfold[S](n, a)
    square_matrix_mul_one_right[S](n, a)
}

/// A successor power entry is the corresponding product-sum entry.
theorem matrix_entry_pow_suc[S: Semiring](n: Nat, a: Matrix[S, n, n], k: Nat, i: Fin[n], j: Fin[n]) {
    matrix_entry[S](n, n, matrix_pow[S](n, a, k.suc), i, j) =
        fin_sum[S](n, matrix_mul_term[S](n, n, n, a, matrix_pow[S](n, a, k), i, j))
} by {
    matrix_pow_suc[S](n, a, k)
    matrix_entry_square_mul[S](n, a, matrix_pow[S](n, a, k), i, j)
}

/// The second power is multiplication by the first-power unfolding.
theorem matrix_pow_two_unfold[S: Semiring](n: Nat, a: Matrix[S, n, n]) {
    matrix_pow[S](n, a, Nat.2) = square_matrix_mul[S](n, a, square_matrix_mul[S](n, a, matrix_one[S](n)))
} by {
    matrix_pow_suc[S](n, a, Nat.1)
    matrix_pow_one_unfold[S](n, a)
}

/// A positive power of the zero matrix is zero.
theorem matrix_pow_zero_matrix_suc[S: Semiring](n: Nat, k: Nat) {
    matrix_pow[S](n, matrix_zero[S](n, n), k.suc) = matrix_zero[S](n, n)
} by {
    define p(exp: Nat) -> Bool {
        matrix_pow[S](n, matrix_zero[S](n, n), exp.suc) = matrix_zero[S](n, n)
    }

    matrix_pow_suc[S](n, matrix_zero[S](n, n), Nat.0)
    square_matrix_mul_zero_left[S](n, matrix_one[S](n))
    matrix_pow[S](n, matrix_zero[S](n, n), Nat.0) = matrix_one[S](n)
    p(Nat.0)

    forall(exp: Nat) {
        if p(exp) {
            matrix_pow_suc[S](n, matrix_zero[S](n, n), exp.suc)
            square_matrix_mul_zero_left[S](n, matrix_pow[S](n, matrix_zero[S](n, n), exp.suc))
            matrix_pow[S](n, matrix_zero[S](n, n), exp.suc.suc) = matrix_zero[S](n, n)
            p(exp.suc)
        }
    }
    p(k)
}

/// The transpose of the zeroth power is the identity matrix.
theorem matrix_transpose_pow_zero[S: Semiring](n: Nat, a: Matrix[S, n, n]) {
    matrix_transpose[S](n, n, matrix_pow[S](n, a, Nat.0)) = matrix_one[S](n)
} by {
    matrix_pow_zero[S](n, a)
    matrix_transpose_one[S](n)
}

/// Over a commutative ring, the transpose of a successor power unfolds as a reversed product.
theorem matrix_transpose_pow_suc_unfold[R: CommRing](n: Nat, a: Matrix[R, n, n], k: Nat) {
    matrix_transpose[R](n, n, matrix_pow[R](n, a, k.suc)) =
        square_matrix_mul[R](n, matrix_transpose[R](n, n, matrix_pow[R](n, a, k)), matrix_transpose[R](n, n, a))
} by {
    matrix_pow_suc[R](n, a, k)
    matrix_transpose_square_mul[R](n, a, matrix_pow[R](n, a, k))
}

/// The successor power of a transpose unfolds by left multiplication with the transpose.
theorem matrix_pow_transpose_suc_unfold[S: Semiring](n: Nat, a: Matrix[S, n, n], k: Nat) {
    matrix_pow[S](n, matrix_transpose[S](n, n, a), k.suc) =
        square_matrix_mul[S](n, matrix_transpose[S](n, n, a), matrix_pow[S](n, matrix_transpose[S](n, n, a), k))
} by {
    matrix_pow_suc[S](n, matrix_transpose[S](n, n, a), k)
}

/// Transposition commutes with the first power of a square matrix.
theorem matrix_transpose_pow_one[S: Semiring](n: Nat, a: Matrix[S, n, n]) {
    matrix_transpose[S](n, n, matrix_pow[S](n, a, Nat.1)) =
        matrix_pow[S](n, matrix_transpose[S](n, n, a), Nat.1)
} by {
    matrix_pow_one[S](n, a)
    matrix_pow_one[S](n, matrix_transpose[S](n, n, a))
}

/// Matrix addition is associative.
theorem matrix_add_semigroup_law[A: AddSemigroup](m: Nat, n: Nat, a: Matrix[A, m, n], b: Matrix[A, m, n], c: Matrix[A, m, n]) {
    Add.add(a, Add.add(b, c)) = Add.add(Add.add(a, b), c)
} by {
    matrix_add_associative[A](m, n, a, b, c)
}

/// Matrices over an additive semigroup form an additive semigroup.
instance Matrix[A: AddSemigroup, m: Nat, n: Nat]: AddSemigroup

/// Matrix addition is commutative.
theorem matrix_add_comm_semigroup_law[A: AddCommSemigroup](m: Nat, n: Nat, a: Matrix[A, m, n], b: Matrix[A, m, n]) {
    Add.add(a, b) = Add.add(b, a)
} by {
    matrix_add_commutative[A](m, n, a, b)
}

/// Matrices over an additive commutative semigroup form an additive commutative semigroup.
instance Matrix[A: AddCommSemigroup, m: Nat, n: Nat]: AddCommSemigroup

/// The zero matrix is a right identity for matrix addition.
theorem matrix_add_monoid_right_law[A: AddMonoid](m: Nat, n: Nat, a: Matrix[A, m, n]) {
    Add.add(a, Zero.0[Matrix[A, m, n]]) = a
} by {
    Zero.0[Matrix[A, m, n]] = matrix_zero[A](m, n)
    matrix_add_identity_right[A](m, n, a)
}

/// The zero matrix is a left identity for matrix addition.
theorem matrix_add_monoid_left_law[A: AddMonoid](m: Nat, n: Nat, a: Matrix[A, m, n]) {
    Add.add(Zero.0[Matrix[A, m, n]], a) = a
} by {
    Zero.0[Matrix[A, m, n]] = matrix_zero[A](m, n)
    matrix_add_identity_left[A](m, n, a)
}

/// Matrices over an additive monoid form an additive monoid.
instance Matrix[A: AddMonoid, m: Nat, n: Nat]: AddMonoid

/// Matrices over an additive commutative monoid form an additive commutative monoid.
instance Matrix[A: AddCommMonoid, m: Nat, n: Nat]: AddCommMonoid

/// Matrix negation gives a right additive inverse.
theorem matrix_add_group_inverse_law[A: AddGroup](m: Nat, n: Nat, a: Matrix[A, m, n]) {
    Add.add(a, Neg.neg(a)) = Zero.0[Matrix[A, m, n]]
} by {
    Zero.0[Matrix[A, m, n]] = matrix_zero[A](m, n)
    matrix_inverse_right[A](m, n, a)
}

/// Matrices over an additive group form an additive group.
instance Matrix[A: AddGroup, m: Nat, n: Nat]: AddGroup

/// Matrices over an additive commutative group form an additive commutative group.
instance Matrix[A: AddCommGroup, m: Nat, n: Nat]: AddCommGroup

/// The entry function of a matrix with two rows exchanged.
define matrix_swap_rows_entry[A](m: Nat, n: Nat, a: Matrix[A, m, n], i: Fin[m], j: Fin[m], r: Fin[m], c: Fin[n]) -> A {
    if r = i {
        matrix_entry[A](m, n, a, j, c)
    } else {
        if r = j {
            matrix_entry[A](m, n, a, i, c)
        } else {
            matrix_entry[A](m, n, a, r, c)
        }
    }
}

/// The matrix obtained by exchanging two of its rows.
define matrix_swap_rows[A](m: Nat, n: Nat, a: Matrix[A, m, n], i: Fin[m], j: Fin[m]) -> Matrix[A, m, n] {
    Matrix[A, m, n].new(matrix_swap_rows_entry[A](m, n, a, i, j))
}

/// The entries of a row-swapped matrix agree with the swap of the original entries.
theorem matrix_entry_swap_rows[A](m: Nat, n: Nat, a: Matrix[A, m, n], i: Fin[m], j: Fin[m], r: Fin[m], c: Fin[n]) {
    matrix_entry[A](m, n, matrix_swap_rows[A](m, n, a, i, j), r, c) =
        matrix_swap_rows_entry[A](m, n, a, i, j, r, c)
} by {
    matrix_entry[A](m, n, matrix_swap_rows[A](m, n, a, i, j), r, c) =
        matrix_swap_rows_entry[A](m, n, a, i, j, r, c)
}
