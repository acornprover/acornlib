from nat import Nat
from nat import lt_trans
from data.fin.fin import Fin, fin_of_nat_suc, fin_of_nat_suc_new_of_lt,
    fin_of_nat_suc_value_of_lt, new_self
from data.basic.logic import and_left, and_right
from list import List, map, length_range, not_unique_implies_duplicate,
    range_contains_all_leq, duplicate_implies_duplicate_idx, map_contains_of_contains,
    map_contains, filter_contained_by_and, filter_contains_and,
    filter_equivalent_to_and, map_length, map_range

numerals Nat

/// The canonical list of all elements of `Fin[n + 1]`, in increasing value order.
define fin_enum_suc(n: Nat) -> List[Fin[n.suc]] {
    map[Nat, Fin[n.suc]](n.suc.range, fin_of_nat_suc(n))
}

/// The successor enumeration has length `n + 1`.
theorem fin_enum_suc_length(n: Nat) {
    fin_enum_suc(n).length = n.suc
} by {
    map_length[Nat, Fin[n.suc]](n.suc.range, fin_of_nat_suc(n))
    length_range(n.suc)
}

/// At each bounded natural index, the successor enumeration contains the represented finite index.
theorem fin_enum_suc_get_idx(n: Nat, k: Nat) {
    k < n.suc implies fin_enum_suc(n).get_idx(k) = Option.some(fin_of_nat_suc(n, k))
} by {
    if k < n.suc {
        map_range[Fin[n.suc]](n.suc, k, fin_of_nat_suc(n))
    }
}

/// The element at a bounded natural index has that natural value.
theorem fin_enum_suc_get_idx_value(n: Nat, k: Nat, i: Fin[n.suc]) {
    k < n.suc and fin_enum_suc(n).get_idx(k) = Option.some(i) implies i.value = k
} by {
    if k < n.suc and fin_enum_suc(n).get_idx(k) = Option.some(i) {
        fin_enum_suc_get_idx(n, k)
        some_injective[Fin[n.suc]](fin_of_nat_suc(n, k), i)
        fin_of_nat_suc(n, k) = i
        fin_of_nat_suc_value_of_lt(n, k)
        i.value = k
    }
}

/// Every element of `Fin[n + 1]` occurs in the successor enumeration.
theorem fin_enum_suc_contains(n: Nat, i: Fin[n.suc]) {
    fin_enum_suc(n).contains(i)
} by {
    i.value < n.suc
    range_contains_all_leq(n.suc)
    n.suc.range.contains(i.value)
    new_self(n.suc, i)
    fin_of_nat_suc_new_of_lt(n, i.value)
    i = fin_of_nat_suc(n, i.value)
    map_contains_of_contains[Nat, Fin[n.suc]](n.suc.range, fin_of_nat_suc(n), i.value)
}

/// Membership in the successor enumeration is automatic for every `Fin[n+1]` value.
theorem fin_enum_suc_contains_iff(n: Nat, i: Fin[n.suc]) {
    fin_enum_suc(n).contains(i) = true
} by {
    fin_enum_suc_contains(n, i)
}

/// Mapping from the successor enumeration detects an element of the domain enumeration.
theorem fin_enum_suc_map_contains_witness[T](n: Nat, f: Fin[n.suc] -> T, y: T) {
    map[Fin[n.suc], T](fin_enum_suc(n), f).contains(y) implies exists(i: Fin[n.suc]) {
        fin_enum_suc(n).contains(i) and f(i) = y
    }
} by {
    if map[Fin[n.suc], T](fin_enum_suc(n), f).contains(y) {
        map_contains[Fin[n.suc], T](fin_enum_suc(n), f, y)
    }
}

/// Filtering the successor enumeration contains exactly those finite indices satisfying the predicate.
theorem fin_enum_suc_filter_contains_iff(n: Nat, p: Fin[n.suc] -> Bool, i: Fin[n.suc]) {
    fin_enum_suc(n).filter(p).contains(i) = p(i)
} by {
    fin_enum_suc_contains(n, i)
    filter_equivalent_to_and[Fin[n.suc]](fin_enum_suc(n), p, i)
}

/// Every element of a filtered successor enumeration satisfies the filter predicate.
theorem fin_enum_suc_filter_contains_pred(n: Nat, p: Fin[n.suc] -> Bool, i: Fin[n.suc]) {
    fin_enum_suc(n).filter(p).contains(i) implies p(i)
} by {
    if fin_enum_suc(n).filter(p).contains(i) {
        filter_contained_by_and[Fin[n.suc]](fin_enum_suc(n), p, i)
    }
}

/// If a finite index satisfies a predicate, it appears in the filtered successor enumeration.
theorem fin_enum_suc_filter_contains_of_pred(n: Nat, p: Fin[n.suc] -> Bool, i: Fin[n.suc]) {
    p(i) implies fin_enum_suc(n).filter(p).contains(i)
} by {
    if p(i) {
        fin_enum_suc_contains(n, i)
        filter_contains_and[Fin[n.suc]](fin_enum_suc(n), p, i)
    }
}

/// The successor enumeration contains no duplicate finite indices.
theorem fin_enum_suc_is_unique(n: Nat) {
    fin_enum_suc(n).is_unique
} by {
    let items = fin_enum_suc(n)
    if not items.is_unique {
        not_unique_implies_duplicate(items)
        let dup: Fin[n.suc] satisfy { items.count(dup) > Nat.1 }
        duplicate_implies_duplicate_idx(items, dup)
        let (i: Nat, j: Nat) satisfy {
            i < j and j < items.length and
            items.get_idx(i) = Option.some(dup) and
            items.get_idx(j) = Option.some(dup)
        }
        let idx_bounds = i < j and j < items.length
        let idx_values = items.get_idx(i) = Option.some(dup) and items.get_idx(j) = Option.some(dup)
        idx_bounds
        idx_values
        and_left(i < j, j < items.length)
        i < j
        and_right(i < j, j < items.length)
        j < items.length
        fin_enum_suc_length(n)
        items.length = n.suc
        j < n.suc
        i < j
        lt_trans(i, j, n.suc)
        i < n.suc
        fin_enum_suc_get_idx(n, i)
        fin_enum_suc_get_idx(n, j)
        items.get_idx(i) = Option.some(fin_of_nat_suc(n, i))
        items.get_idx(j) = Option.some(fin_of_nat_suc(n, j))
        items.get_idx(i) = Option.some(dup)
        items.get_idx(j) = Option.some(dup)
        Option.some(fin_of_nat_suc(n, i)) = Option.some(dup)
        Option.some(fin_of_nat_suc(n, j)) = Option.some(dup)
        some_injective[Fin[n.suc]](fin_of_nat_suc(n, i), dup)
        some_injective[Fin[n.suc]](fin_of_nat_suc(n, j), dup)
        fin_of_nat_suc(n, i) = dup
        fin_of_nat_suc(n, j) = dup
        fin_of_nat_suc_value_of_lt(n, i)
        fin_of_nat_suc_value_of_lt(n, j)
        dup.value = i
        dup.value = j
        i = j
        false
    }
}
