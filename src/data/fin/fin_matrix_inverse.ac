/// Two-by-two matrix inverse theory: the adjugate identity, the inverse
/// formula `inv(A) = (1/det(A)) * adj(A)`, uniqueness of the inverse,
/// multiplicativity of the inverse, the invertibility criterion
/// `det(A) != 0` iff `A` is invertible, and Cramer's rule.
///
/// The two-by-two case is the landing case for the general adjugate identity
/// `A * adj(A) = det(A) * I`: the general statement needs a determinant that
/// is known to agree with the Laplace expansion along every row and to be
/// alternating, which the library's determinant-list machinery does not yet
/// provide (see the file-level notes in `fin_matrix_det.ac`).  The adjugate
/// used here is the general one from `fin_matrix.ac`, instantiated at
/// dimension two with the canonical determinant on one-by-one minors.
///
/// Rank and nullity are not developed here: the library has no notion of the
/// dimension of a subspace (no basis theory beyond singleton linear
/// independence, in `algebra/module/vector_space.ac`), so the rank-nullity
/// theorem and the inequality `rank(AB) <= rank(A)` await a dimension theory
/// for finite-dimensional vector spaces.
///
/// The matrix-valued helpers (`matrix2_adjugate`, `matrix2_inv`,
/// `matrix2_scalar_one`, `matrix2_smul`, `matrix2_replace_col`) are defined in
/// `algebra/matrix_algebra.ac` and imported here: the certificate serializer
/// currently cannot round-trip claims that apply a function whose signature
/// contains a fixed-size `Matrix[F, ...]` type, so every helper is
/// parameterized by an ambient size `n` and used at `n = 0` for the
/// two-by-two case.

from nat import Nat
from semiring import Semiring
from algebra.field.field import Field, inverse_one, mul_inverse_right, mul_inverse_left,
    field_mul_nonzero
from algebra.ring.ring import Ring, alternating_sign, alternating_sign_zero,
    alternating_sign_suc, mul_neg_left, mul_neg_right, mul_neg_neg, mul_zero_left,
    mul_sub_left, mul_sub_right
from comm_ring import CommRing
from data.fin.fin import Fin, fin_zero, fin_succ, fin_cast_suc, fin_zero_value,
    fin_succ_value, fin_cast_suc_value, fin_skip, fin_skip_value, fin_skip_value_eq,
    ext, value_lt_bound
from data.fin.fin_matrix import Matrix, matrix_entry, matrix_ext, matrix_one,
    square_matrix_mul, matrix_entry_one_diag, matrix_entry_one_off_diag,
    matrix_minor, matrix_entry_minor, matrix_cofactor_with, matrix_cofactor_with_eq,
    matrix_cofactor_sign
from data.fin.fin_matrix_det import matrix_det_suc, matrix_det_suc_zero_eq_entry
from data.fin.fin_matrix_eigen import matrix_apply, matrix_apply_eq, matrix_apply_term,
    matrix_apply_pointwise_eq
from data.fin.fin_sum import fin_sum
from algebra.add_comm_monoid_rearrange import add_swap_inner
from algebra.matrix_algebra import det2_entry, det2_entry_expanded, fin_sum_two,
    matrix_entry_mul_two, square_matrix_mul_assoc_two, square_matrix_mul_one_right_two,
    square_matrix_mul_one_left_two, matrix_apply_mul_two, matrix_apply_vec,
    fin_two_zero_ne_one, fin_two_one_ne_zero, matrix2_entry_00, matrix2_entry_01,
    matrix2_entry_10, matrix2_entry_11, matrix2_det, matrix2_adjugate_entry,
    matrix2_adjugate, matrix2_scalar_one_entry, matrix2_inv_entry, matrix2_inv,
    matrix2_replace_col,
    matrix2_replace_col_entry, inv2_entry_00, inv2_entry_01, inv2_entry_10,
    inv2_entry_11

// ---------------------------------------------------------------------------
// The two elements of `Fin[2]`
// ---------------------------------------------------------------------------

/// The first index of `Fin[2]` has value zero.
theorem fin_two_zero_value_two {
    fin_zero(Nat.0.suc).value = Nat.0
} by {
    fin_zero_value(Nat.0.suc)
}

/// The second index of `Fin[2]` has value one.
theorem fin_two_one_value_two {
    fin_succ(Nat.0.suc, fin_zero(Nat.0)).value = Nat.0.suc
} by {
    fin_succ_value(Nat.0.suc, fin_zero(Nat.0))
}

/// Skipping the first index of `Fin[2]` past the first index of `Fin[1]`
/// lands on the second index.
theorem fin_skip_first_two {
    fin_skip(Nat.0.suc, fin_zero(Nat.0.suc), fin_zero(Nat.0)) = fin_succ(Nat.0.suc, fin_zero(Nat.0))
} by {
    fin_skip_value_eq(Nat.0.suc, fin_zero(Nat.0.suc), fin_zero(Nat.0))
    fin_zero_value(Nat.0.suc)
    fin_zero_value(Nat.0)
    fin_skip_value(Nat.0.suc, fin_zero(Nat.0.suc), fin_zero(Nat.0)) = Nat.0.suc
    fin_succ_value(Nat.0.suc, fin_zero(Nat.0))
    fin_succ(Nat.0.suc, fin_zero(Nat.0)).value = Nat.0.suc
    ext(Nat.0.suc.suc, fin_skip(Nat.0.suc, fin_zero(Nat.0.suc), fin_zero(Nat.0)), fin_succ(Nat.0.suc, fin_zero(Nat.0)))
    fin_skip(Nat.0.suc, fin_zero(Nat.0.suc), fin_zero(Nat.0)) = fin_succ(Nat.0.suc, fin_zero(Nat.0))
}

/// Skipping the second index of `Fin[2]` past the first index of `Fin[1]`
/// lands on the cast of the first index.
theorem fin_skip_second_two {
    fin_skip(Nat.0.suc, fin_succ(Nat.0.suc, fin_zero(Nat.0)), fin_zero(Nat.0)) =
        fin_cast_suc(Nat.0.suc, fin_zero(Nat.0))
} by {
    fin_skip_value_eq(Nat.0.suc, fin_succ(Nat.0.suc, fin_zero(Nat.0)), fin_zero(Nat.0))
    fin_two_one_value_two
    fin_zero_value(Nat.0)
    fin_skip_value(Nat.0.suc, fin_succ(Nat.0.suc, fin_zero(Nat.0)), fin_zero(Nat.0)) = Nat.0
    fin_cast_suc_value(Nat.0.suc, fin_zero(Nat.0))
    fin_cast_suc(Nat.0.suc, fin_zero(Nat.0)).value = Nat.0
    ext(Nat.0.suc.suc, fin_skip(Nat.0.suc, fin_succ(Nat.0.suc, fin_zero(Nat.0)), fin_zero(Nat.0)), fin_cast_suc(Nat.0.suc, fin_zero(Nat.0)))
    fin_skip(Nat.0.suc, fin_succ(Nat.0.suc, fin_zero(Nat.0)), fin_zero(Nat.0)) =
        fin_cast_suc(Nat.0.suc, fin_zero(Nat.0))
}

/// The cast of the first index of `Fin[1]` into `Fin[2]` is the first index.
theorem fin_cast_suc_first_two {
    fin_cast_suc(Nat.0.suc, fin_zero(Nat.0)) = fin_zero(Nat.0.suc)
} by {
    fin_cast_suc_value(Nat.0.suc, fin_zero(Nat.0))
    fin_zero_value(Nat.0.suc)
    fin_cast_suc(Nat.0.suc, fin_zero(Nat.0)).value = fin_zero(Nat.0.suc).value
    ext(Nat.0.suc.suc, fin_cast_suc(Nat.0.suc, fin_zero(Nat.0)), fin_zero(Nat.0.suc))
    fin_cast_suc(Nat.0.suc, fin_zero(Nat.0)) = fin_zero(Nat.0.suc)
}

/// An index of `Fin[2]` is either the first or the second.
theorem fin_two_index_cases(i: Fin[Nat.0.suc.suc]) {
    i.value = Nat.0 or i.value = Nat.0.suc
} by {
    value_lt_bound(Nat.0.suc.suc, i)
    i.value < Nat.0.suc.suc
    if i.value = Nat.0 {
        i.value = Nat.0 or i.value = Nat.0.suc
    } else {
        i.value = Nat.0.suc
        i.value = Nat.0 or i.value = Nat.0.suc
    }
}

// ---------------------------------------------------------------------------
// The determinant and the adjugate entries of a two-by-two matrix
// ---------------------------------------------------------------------------

/// The two-by-two determinant unfolds to the entry formula.
theorem matrix2_det_unfold[F: Ring](a: Matrix[F, Nat.0.suc.suc, Nat.0.suc.suc]) {
    matrix2_det[F](Nat.0, a) =
        matrix2_entry_00[F](Nat.0, a) * matrix2_entry_11[F](Nat.0, a) -
        matrix2_entry_01[F](Nat.0, a) * matrix2_entry_10[F](Nat.0, a)
} by {
    matrix2_det[F](Nat.0, a) =
        matrix2_entry_00[F](Nat.0, a) * matrix2_entry_11[F](Nat.0, a) -
        matrix2_entry_01[F](Nat.0, a) * matrix2_entry_10[F](Nat.0, a)
}

/// An entry of the two-by-two adjugate matrix is the corresponding adjugate
/// entry.
theorem matrix2_adjugate_matrix_entry[F: Field](
    a: Matrix[F, Nat.0.suc.suc, Nat.0.suc.suc], i: Fin[Nat.0.suc.suc], j: Fin[Nat.0.suc.suc]
) {
    matrix_entry[F](Nat.0.suc.suc, Nat.0.suc.suc, matrix2_adjugate[F](Nat.0, a), i, j) =
        matrix2_adjugate_entry[F](Nat.0, a, i, j)
} by {
    matrix_entry[F](Nat.0.suc.suc, Nat.0.suc.suc, matrix2_adjugate[F](Nat.0, a), i, j) =
        matrix2_adjugate_entry[F](Nat.0, a, i, j)
}

/// The upper-left adjugate entry is the lower-right entry of the matrix.
theorem matrix2_adjugate_entry_00[F: Field](a: Matrix[F, Nat.0.suc.suc, Nat.0.suc.suc]) {
    matrix2_adjugate_entry[F](Nat.0, a, fin_zero(Nat.0.suc), fin_zero(Nat.0.suc)) = matrix2_entry_11[F](Nat.0, a)
} by {
    matrix2_adjugate_entry[F](Nat.0, a, fin_zero(Nat.0.suc), fin_zero(Nat.0.suc)) =
        matrix_cofactor_with[F](Nat.0.suc, matrix_det_suc[F](Nat.0), a, fin_zero(Nat.0.suc), fin_zero(Nat.0.suc))
    matrix_cofactor_with_eq[F](Nat.0.suc, matrix_det_suc[F](Nat.0), a, fin_zero(Nat.0.suc), fin_zero(Nat.0.suc))
    matrix2_adjugate_entry[F](Nat.0, a, fin_zero(Nat.0.suc), fin_zero(Nat.0.suc)) =
        matrix_cofactor_sign[F](Nat.0.suc.suc, fin_zero(Nat.0.suc), fin_zero(Nat.0.suc)) *
            matrix_det_suc[F](Nat.0, matrix_minor[F](Nat.0.suc, a, fin_zero(Nat.0.suc), fin_zero(Nat.0.suc)))
    fin_two_zero_value_two
    matrix_cofactor_sign[F](Nat.0.suc.suc, fin_zero(Nat.0.suc), fin_zero(Nat.0.suc)) =
        alternating_sign[F](Nat.0 + Nat.0)
    alternating_sign_zero[F]
    alternating_sign[F](Nat.0) = F.1
    matrix_cofactor_sign[F](Nat.0.suc.suc, fin_zero(Nat.0.suc), fin_zero(Nat.0.suc)) = F.1
    matrix_det_suc_zero_eq_entry[F](Nat.0, matrix_minor[F](Nat.0.suc, a, fin_zero(Nat.0.suc), fin_zero(Nat.0.suc)))
    matrix_det_suc[F](Nat.0, matrix_minor[F](Nat.0.suc, a, fin_zero(Nat.0.suc), fin_zero(Nat.0.suc))) =
        matrix_entry[F](Nat.0.suc, Nat.0.suc, matrix_minor[F](Nat.0.suc, a, fin_zero(Nat.0.suc), fin_zero(Nat.0.suc)), fin_zero(Nat.0), fin_zero(Nat.0))
    matrix_entry_minor[F](Nat.0.suc, a, fin_zero(Nat.0.suc), fin_zero(Nat.0.suc), fin_zero(Nat.0), fin_zero(Nat.0))
    matrix_entry[F](Nat.0.suc, Nat.0.suc, matrix_minor[F](Nat.0.suc, a, fin_zero(Nat.0.suc), fin_zero(Nat.0.suc)), fin_zero(Nat.0), fin_zero(Nat.0)) =
        matrix_entry[F](Nat.0.suc.suc, Nat.0.suc.suc, a, fin_skip(Nat.0.suc, fin_zero(Nat.0.suc), fin_zero(Nat.0)), fin_skip(Nat.0.suc, fin_zero(Nat.0.suc), fin_zero(Nat.0)))
    fin_skip_first_two
    matrix_entry[F](Nat.0.suc.suc, Nat.0.suc.suc, a, fin_succ(Nat.0.suc, fin_zero(Nat.0)), fin_succ(Nat.0.suc, fin_zero(Nat.0))) =
        matrix2_entry_11[F](Nat.0, a)
    matrix_det_suc[F](Nat.0, matrix_minor[F](Nat.0.suc, a, fin_zero(Nat.0.suc), fin_zero(Nat.0.suc))) =
        matrix2_entry_11[F](Nat.0, a)
    F.1 * matrix_det_suc[F](Nat.0, matrix_minor[F](Nat.0.suc, a, fin_zero(Nat.0.suc), fin_zero(Nat.0.suc))) =
        matrix2_entry_11[F](Nat.0, a)
    matrix2_adjugate_entry[F](Nat.0, a, fin_zero(Nat.0.suc), fin_zero(Nat.0.suc)) = matrix2_entry_11[F](Nat.0, a)
}

/// The upper-right adjugate entry is the negated upper-right entry of the
/// matrix.
theorem matrix2_adjugate_entry_01[F: Field](a: Matrix[F, Nat.0.suc.suc, Nat.0.suc.suc]) {
    matrix2_adjugate_entry[F](Nat.0, a, fin_zero(Nat.0.suc), fin_succ(Nat.0.suc, fin_zero(Nat.0))) =
        -matrix2_entry_01[F](Nat.0, a)
} by {
    matrix2_adjugate_entry[F](Nat.0, a, fin_zero(Nat.0.suc), fin_succ(Nat.0.suc, fin_zero(Nat.0))) =
        matrix_cofactor_with[F](Nat.0.suc, matrix_det_suc[F](Nat.0), a, fin_succ(Nat.0.suc, fin_zero(Nat.0)), fin_zero(Nat.0.suc))
    matrix_cofactor_with_eq[F](Nat.0.suc, matrix_det_suc[F](Nat.0), a, fin_succ(Nat.0.suc, fin_zero(Nat.0)), fin_zero(Nat.0.suc))
    matrix2_adjugate_entry[F](Nat.0, a, fin_zero(Nat.0.suc), fin_succ(Nat.0.suc, fin_zero(Nat.0))) =
        matrix_cofactor_sign[F](Nat.0.suc.suc, fin_succ(Nat.0.suc, fin_zero(Nat.0)), fin_zero(Nat.0.suc)) *
            matrix_det_suc[F](Nat.0, matrix_minor[F](Nat.0.suc, a, fin_succ(Nat.0.suc, fin_zero(Nat.0)), fin_zero(Nat.0.suc)))
    fin_two_one_value_two
    fin_two_zero_value_two
    matrix_cofactor_sign[F](Nat.0.suc.suc, fin_succ(Nat.0.suc, fin_zero(Nat.0)), fin_zero(Nat.0.suc)) =
        alternating_sign[F](Nat.0.suc + Nat.0)
    alternating_sign_suc[F](Nat.0)
    alternating_sign[F](Nat.0.suc) = -alternating_sign[F](Nat.0)
    alternating_sign_zero[F]
    alternating_sign[F](Nat.0) = F.1
    -alternating_sign[F](Nat.0) = -F.1
    alternating_sign[F](Nat.0.suc) = -F.1
    matrix_cofactor_sign[F](Nat.0.suc.suc, fin_succ(Nat.0.suc, fin_zero(Nat.0)), fin_zero(Nat.0.suc)) = -F.1
    matrix_det_suc_zero_eq_entry[F](Nat.0, matrix_minor[F](Nat.0.suc, a, fin_succ(Nat.0.suc, fin_zero(Nat.0)), fin_zero(Nat.0.suc)))
    matrix_det_suc[F](Nat.0, matrix_minor[F](Nat.0.suc, a, fin_succ(Nat.0.suc, fin_zero(Nat.0)), fin_zero(Nat.0.suc))) =
        matrix_entry[F](Nat.0.suc, Nat.0.suc, matrix_minor[F](Nat.0.suc, a, fin_succ(Nat.0.suc, fin_zero(Nat.0)), fin_zero(Nat.0.suc)), fin_zero(Nat.0), fin_zero(Nat.0))
    matrix_entry_minor[F](Nat.0.suc, a, fin_succ(Nat.0.suc, fin_zero(Nat.0)), fin_zero(Nat.0.suc), fin_zero(Nat.0), fin_zero(Nat.0))
    matrix_entry[F](Nat.0.suc, Nat.0.suc, matrix_minor[F](Nat.0.suc, a, fin_succ(Nat.0.suc, fin_zero(Nat.0)), fin_zero(Nat.0.suc)), fin_zero(Nat.0), fin_zero(Nat.0)) =
        matrix_entry[F](Nat.0.suc.suc, Nat.0.suc.suc, a, fin_skip(Nat.0.suc, fin_succ(Nat.0.suc, fin_zero(Nat.0)), fin_zero(Nat.0)), fin_skip(Nat.0.suc, fin_zero(Nat.0.suc), fin_zero(Nat.0)))
    fin_skip_second_two
    fin_skip_first_two
    fin_cast_suc_first_two
    matrix_entry[F](Nat.0.suc.suc, Nat.0.suc.suc, a, fin_zero(Nat.0.suc), fin_succ(Nat.0.suc, fin_zero(Nat.0))) =
        matrix2_entry_01[F](Nat.0, a)
    matrix_det_suc[F](Nat.0, matrix_minor[F](Nat.0.suc, a, fin_succ(Nat.0.suc, fin_zero(Nat.0)), fin_zero(Nat.0.suc))) =
        matrix2_entry_01[F](Nat.0, a)
    -F.1 * matrix_det_suc[F](Nat.0, matrix_minor[F](Nat.0.suc, a, fin_succ(Nat.0.suc, fin_zero(Nat.0)), fin_zero(Nat.0.suc))) =
        -matrix2_entry_01[F](Nat.0, a)
    matrix2_adjugate_entry[F](Nat.0, a, fin_zero(Nat.0.suc), fin_succ(Nat.0.suc, fin_zero(Nat.0))) =
        -matrix2_entry_01[F](Nat.0, a)
}

/// The lower-left adjugate entry is the negated lower-left entry of the
/// matrix.
theorem matrix2_adjugate_entry_10[F: Field](a: Matrix[F, Nat.0.suc.suc, Nat.0.suc.suc]) {
    matrix2_adjugate_entry[F](Nat.0, a, fin_succ(Nat.0.suc, fin_zero(Nat.0)), fin_zero(Nat.0.suc)) =
        -matrix2_entry_10[F](Nat.0, a)
} by {
    matrix2_adjugate_entry[F](Nat.0, a, fin_succ(Nat.0.suc, fin_zero(Nat.0)), fin_zero(Nat.0.suc)) =
        matrix_cofactor_with[F](Nat.0.suc, matrix_det_suc[F](Nat.0), a, fin_zero(Nat.0.suc), fin_succ(Nat.0.suc, fin_zero(Nat.0)))
    matrix_cofactor_with_eq[F](Nat.0.suc, matrix_det_suc[F](Nat.0), a, fin_zero(Nat.0.suc), fin_succ(Nat.0.suc, fin_zero(Nat.0)))
    matrix2_adjugate_entry[F](Nat.0, a, fin_succ(Nat.0.suc, fin_zero(Nat.0)), fin_zero(Nat.0.suc)) =
        matrix_cofactor_sign[F](Nat.0.suc.suc, fin_zero(Nat.0.suc), fin_succ(Nat.0.suc, fin_zero(Nat.0))) *
            matrix_det_suc[F](Nat.0, matrix_minor[F](Nat.0.suc, a, fin_zero(Nat.0.suc), fin_succ(Nat.0.suc, fin_zero(Nat.0))))
    fin_two_zero_value_two
    fin_two_one_value_two
    matrix_cofactor_sign[F](Nat.0.suc.suc, fin_zero(Nat.0.suc), fin_succ(Nat.0.suc, fin_zero(Nat.0))) =
        alternating_sign[F](Nat.0 + Nat.0.suc)
    alternating_sign_suc[F](Nat.0)
    alternating_sign[F](Nat.0.suc) = -alternating_sign[F](Nat.0)
    alternating_sign_zero[F]
    alternating_sign[F](Nat.0) = F.1
    -alternating_sign[F](Nat.0) = -F.1
    alternating_sign[F](Nat.0.suc) = -F.1
    matrix_cofactor_sign[F](Nat.0.suc.suc, fin_zero(Nat.0.suc), fin_succ(Nat.0.suc, fin_zero(Nat.0))) = -F.1
    matrix_det_suc_zero_eq_entry[F](Nat.0, matrix_minor[F](Nat.0.suc, a, fin_zero(Nat.0.suc), fin_succ(Nat.0.suc, fin_zero(Nat.0))))
    matrix_det_suc[F](Nat.0, matrix_minor[F](Nat.0.suc, a, fin_zero(Nat.0.suc), fin_succ(Nat.0.suc, fin_zero(Nat.0)))) =
        matrix_entry[F](Nat.0.suc, Nat.0.suc, matrix_minor[F](Nat.0.suc, a, fin_zero(Nat.0.suc), fin_succ(Nat.0.suc, fin_zero(Nat.0))), fin_zero(Nat.0), fin_zero(Nat.0))
    matrix_entry_minor[F](Nat.0.suc, a, fin_zero(Nat.0.suc), fin_succ(Nat.0.suc, fin_zero(Nat.0)), fin_zero(Nat.0), fin_zero(Nat.0))
    matrix_entry[F](Nat.0.suc, Nat.0.suc, matrix_minor[F](Nat.0.suc, a, fin_zero(Nat.0.suc), fin_succ(Nat.0.suc, fin_zero(Nat.0))), fin_zero(Nat.0), fin_zero(Nat.0)) =
        matrix_entry[F](Nat.0.suc.suc, Nat.0.suc.suc, a, fin_skip(Nat.0.suc, fin_zero(Nat.0.suc), fin_zero(Nat.0)), fin_skip(Nat.0.suc, fin_succ(Nat.0.suc, fin_zero(Nat.0)), fin_zero(Nat.0)))
    fin_skip_first_two
    fin_skip_second_two
    fin_cast_suc_first_two
    matrix_entry[F](Nat.0.suc.suc, Nat.0.suc.suc, a, fin_succ(Nat.0.suc, fin_zero(Nat.0)), fin_zero(Nat.0.suc)) =
        matrix2_entry_10[F](Nat.0, a)
    matrix_det_suc[F](Nat.0, matrix_minor[F](Nat.0.suc, a, fin_zero(Nat.0.suc), fin_succ(Nat.0.suc, fin_zero(Nat.0)))) =
        matrix2_entry_10[F](Nat.0, a)
    -F.1 * matrix_det_suc[F](Nat.0, matrix_minor[F](Nat.0.suc, a, fin_zero(Nat.0.suc), fin_succ(Nat.0.suc, fin_zero(Nat.0)))) =
        -matrix2_entry_10[F](Nat.0, a)
    matrix2_adjugate_entry[F](Nat.0, a, fin_succ(Nat.0.suc, fin_zero(Nat.0)), fin_zero(Nat.0.suc)) =
        -matrix2_entry_10[F](Nat.0, a)
}

/// The lower-right adjugate entry is the upper-left entry of the matrix.
theorem matrix2_adjugate_entry_11[F: Field](a: Matrix[F, Nat.0.suc.suc, Nat.0.suc.suc]) {
    matrix2_adjugate_entry[F](Nat.0, a, fin_succ(Nat.0.suc, fin_zero(Nat.0)), fin_succ(Nat.0.suc, fin_zero(Nat.0))) =
        matrix2_entry_00[F](Nat.0, a)
} by {
    matrix2_adjugate_entry[F](Nat.0, a, fin_succ(Nat.0.suc, fin_zero(Nat.0)), fin_succ(Nat.0.suc, fin_zero(Nat.0))) =
        matrix_cofactor_with[F](Nat.0.suc, matrix_det_suc[F](Nat.0), a, fin_succ(Nat.0.suc, fin_zero(Nat.0)), fin_succ(Nat.0.suc, fin_zero(Nat.0)))
    matrix_cofactor_with_eq[F](Nat.0.suc, matrix_det_suc[F](Nat.0), a, fin_succ(Nat.0.suc, fin_zero(Nat.0)), fin_succ(Nat.0.suc, fin_zero(Nat.0)))
    matrix2_adjugate_entry[F](Nat.0, a, fin_succ(Nat.0.suc, fin_zero(Nat.0)), fin_succ(Nat.0.suc, fin_zero(Nat.0))) =
        matrix_cofactor_sign[F](Nat.0.suc.suc, fin_succ(Nat.0.suc, fin_zero(Nat.0)), fin_succ(Nat.0.suc, fin_zero(Nat.0))) *
            matrix_det_suc[F](Nat.0, matrix_minor[F](Nat.0.suc, a, fin_succ(Nat.0.suc, fin_zero(Nat.0)), fin_succ(Nat.0.suc, fin_zero(Nat.0))))
    fin_two_one_value_two
    matrix_cofactor_sign[F](Nat.0.suc.suc, fin_succ(Nat.0.suc, fin_zero(Nat.0)), fin_succ(Nat.0.suc, fin_zero(Nat.0))) =
        alternating_sign[F](Nat.0.suc + Nat.0.suc)
    alternating_sign_suc[F](Nat.0.suc)
    alternating_sign[F](Nat.0.suc.suc) = -alternating_sign[F](Nat.0.suc)
    alternating_sign_suc[F](Nat.0)
    alternating_sign[F](Nat.0.suc) = -alternating_sign[F](Nat.0)
    alternating_sign_zero[F]
    alternating_sign[F](Nat.0) = F.1
    -alternating_sign[F](Nat.0) = -F.1
    alternating_sign[F](Nat.0.suc) = -F.1
    -(-F.1) = F.1
    -alternating_sign[F](Nat.0.suc) = F.1
    alternating_sign[F](Nat.0.suc.suc) = F.1
    matrix_cofactor_sign[F](Nat.0.suc.suc, fin_succ(Nat.0.suc, fin_zero(Nat.0)), fin_succ(Nat.0.suc, fin_zero(Nat.0))) = F.1
    matrix_det_suc_zero_eq_entry[F](Nat.0, matrix_minor[F](Nat.0.suc, a, fin_succ(Nat.0.suc, fin_zero(Nat.0)), fin_succ(Nat.0.suc, fin_zero(Nat.0))))
    matrix_det_suc[F](Nat.0, matrix_minor[F](Nat.0.suc, a, fin_succ(Nat.0.suc, fin_zero(Nat.0)), fin_succ(Nat.0.suc, fin_zero(Nat.0)))) =
        matrix_entry[F](Nat.0.suc, Nat.0.suc, matrix_minor[F](Nat.0.suc, a, fin_succ(Nat.0.suc, fin_zero(Nat.0)), fin_succ(Nat.0.suc, fin_zero(Nat.0))), fin_zero(Nat.0), fin_zero(Nat.0))
    matrix_entry_minor[F](Nat.0.suc, a, fin_succ(Nat.0.suc, fin_zero(Nat.0)), fin_succ(Nat.0.suc, fin_zero(Nat.0)), fin_zero(Nat.0), fin_zero(Nat.0))
    matrix_entry[F](Nat.0.suc, Nat.0.suc, matrix_minor[F](Nat.0.suc, a, fin_succ(Nat.0.suc, fin_zero(Nat.0)), fin_succ(Nat.0.suc, fin_zero(Nat.0))), fin_zero(Nat.0), fin_zero(Nat.0)) =
        matrix_entry[F](Nat.0.suc.suc, Nat.0.suc.suc, a, fin_skip(Nat.0.suc, fin_succ(Nat.0.suc, fin_zero(Nat.0)), fin_zero(Nat.0)), fin_skip(Nat.0.suc, fin_succ(Nat.0.suc, fin_zero(Nat.0)), fin_zero(Nat.0)))
    fin_skip_second_two
    fin_cast_suc_first_two
    matrix_entry[F](Nat.0.suc.suc, Nat.0.suc.suc, a, fin_zero(Nat.0.suc), fin_zero(Nat.0.suc)) =
        matrix2_entry_00[F](Nat.0, a)
    matrix_det_suc[F](Nat.0, matrix_minor[F](Nat.0.suc, a, fin_succ(Nat.0.suc, fin_zero(Nat.0)), fin_succ(Nat.0.suc, fin_zero(Nat.0)))) =
        matrix2_entry_00[F](Nat.0, a)
    F.1 * matrix_det_suc[F](Nat.0, matrix_minor[F](Nat.0.suc, a, fin_succ(Nat.0.suc, fin_zero(Nat.0)), fin_succ(Nat.0.suc, fin_zero(Nat.0)))) =
        matrix2_entry_00[F](Nat.0, a)
    matrix2_adjugate_entry[F](Nat.0, a, fin_succ(Nat.0.suc, fin_zero(Nat.0)), fin_succ(Nat.0.suc, fin_zero(Nat.0))) =
        matrix2_entry_00[F](Nat.0, a)
}

// ---------------------------------------------------------------------------
// Entries of the product of two-by-two matrices
// ---------------------------------------------------------------------------

/// An entry of the product of two two-by-two matrices is the two-term sum of
/// products of entries.
theorem matrix2_mul_entry[F: Ring](a: Matrix[F, Nat.0.suc.suc, Nat.0.suc.suc], b: Matrix[F, Nat.0.suc.suc, Nat.0.suc.suc], i: Fin[Nat.0.suc.suc], j: Fin[Nat.0.suc.suc]) {
    matrix_entry[F](Nat.0.suc.suc, Nat.0.suc.suc, square_matrix_mul[F](Nat.0.suc.suc, a, b), i, j) =
        matrix_entry[F](Nat.0.suc.suc, Nat.0.suc.suc, a, i, fin_zero(Nat.0.suc)) *
            matrix_entry[F](Nat.0.suc.suc, Nat.0.suc.suc, b, fin_zero(Nat.0.suc), j) +
        matrix_entry[F](Nat.0.suc.suc, Nat.0.suc.suc, a, i, fin_succ(Nat.0.suc, fin_zero(Nat.0))) *
            matrix_entry[F](Nat.0.suc.suc, Nat.0.suc.suc, b, fin_succ(Nat.0.suc, fin_zero(Nat.0)), j)
} by {
    matrix_entry_mul_two[F](a, b, i, j)
    matrix_entry[F](Nat.0.suc.suc, Nat.0.suc.suc, square_matrix_mul[F](Nat.0.suc.suc, a, b), i, j) =
        matrix_entry[F](Nat.0.suc.suc, Nat.0.suc.suc, a, i, fin_zero(Nat.0.suc)) *
            matrix_entry[F](Nat.0.suc.suc, Nat.0.suc.suc, b, fin_zero(Nat.0.suc), j) +
        matrix_entry[F](Nat.0.suc.suc, Nat.0.suc.suc, a, i, fin_succ(Nat.0.suc, fin_zero(Nat.0))) *
            matrix_entry[F](Nat.0.suc.suc, Nat.0.suc.suc, b, fin_succ(Nat.0.suc, fin_zero(Nat.0)), j)
}

/// The upper-left entry of the product of two two-by-two matrices.
theorem matrix2_mul_entry_00[F: Ring](a: Matrix[F, Nat.0.suc.suc, Nat.0.suc.suc], b: Matrix[F, Nat.0.suc.suc, Nat.0.suc.suc]) {
    matrix_entry[F](Nat.0.suc.suc, Nat.0.suc.suc, square_matrix_mul[F](Nat.0.suc.suc, a, b), fin_zero(Nat.0.suc), fin_zero(Nat.0.suc)) =
        matrix2_entry_00[F](Nat.0, a) * matrix2_entry_00[F](Nat.0, b) + matrix2_entry_01[F](Nat.0, a) * matrix2_entry_10[F](Nat.0, b)
} by {
    matrix_entry_mul_two[F](a, b, fin_zero(Nat.0.suc), fin_zero(Nat.0.suc))
    matrix_entry[F](Nat.0.suc.suc, Nat.0.suc.suc, square_matrix_mul[F](Nat.0.suc.suc, a, b), fin_zero(Nat.0.suc), fin_zero(Nat.0.suc)) =
        matrix_entry[F](Nat.0.suc.suc, Nat.0.suc.suc, a, fin_zero(Nat.0.suc), fin_zero(Nat.0.suc)) *
            matrix_entry[F](Nat.0.suc.suc, Nat.0.suc.suc, b, fin_zero(Nat.0.suc), fin_zero(Nat.0.suc)) +
        matrix_entry[F](Nat.0.suc.suc, Nat.0.suc.suc, a, fin_zero(Nat.0.suc), fin_succ(Nat.0.suc, fin_zero(Nat.0))) *
            matrix_entry[F](Nat.0.suc.suc, Nat.0.suc.suc, b, fin_succ(Nat.0.suc, fin_zero(Nat.0)), fin_zero(Nat.0.suc))
    matrix_entry[F](Nat.0.suc.suc, Nat.0.suc.suc, square_matrix_mul[F](Nat.0.suc.suc, a, b), fin_zero(Nat.0.suc), fin_zero(Nat.0.suc)) =
        matrix2_entry_00[F](Nat.0, a) * matrix2_entry_00[F](Nat.0, b) + matrix2_entry_01[F](Nat.0, a) * matrix2_entry_10[F](Nat.0, b)
}

/// The upper-right entry of the product of two two-by-two matrices.
theorem matrix2_mul_entry_01[F: Ring](a: Matrix[F, Nat.0.suc.suc, Nat.0.suc.suc], b: Matrix[F, Nat.0.suc.suc, Nat.0.suc.suc]) {
    matrix_entry[F](Nat.0.suc.suc, Nat.0.suc.suc, square_matrix_mul[F](Nat.0.suc.suc, a, b), fin_zero(Nat.0.suc), fin_succ(Nat.0.suc, fin_zero(Nat.0))) =
        matrix2_entry_00[F](Nat.0, a) * matrix2_entry_01[F](Nat.0, b) + matrix2_entry_01[F](Nat.0, a) * matrix2_entry_11[F](Nat.0, b)
} by {
    matrix_entry_mul_two[F](a, b, fin_zero(Nat.0.suc), fin_succ(Nat.0.suc, fin_zero(Nat.0)))
    matrix_entry[F](Nat.0.suc.suc, Nat.0.suc.suc, square_matrix_mul[F](Nat.0.suc.suc, a, b), fin_zero(Nat.0.suc), fin_succ(Nat.0.suc, fin_zero(Nat.0))) =
        matrix_entry[F](Nat.0.suc.suc, Nat.0.suc.suc, a, fin_zero(Nat.0.suc), fin_zero(Nat.0.suc)) *
            matrix_entry[F](Nat.0.suc.suc, Nat.0.suc.suc, b, fin_zero(Nat.0.suc), fin_succ(Nat.0.suc, fin_zero(Nat.0))) +
        matrix_entry[F](Nat.0.suc.suc, Nat.0.suc.suc, a, fin_zero(Nat.0.suc), fin_succ(Nat.0.suc, fin_zero(Nat.0))) *
            matrix_entry[F](Nat.0.suc.suc, Nat.0.suc.suc, b, fin_succ(Nat.0.suc, fin_zero(Nat.0)), fin_succ(Nat.0.suc, fin_zero(Nat.0)))
    matrix_entry[F](Nat.0.suc.suc, Nat.0.suc.suc, square_matrix_mul[F](Nat.0.suc.suc, a, b), fin_zero(Nat.0.suc), fin_succ(Nat.0.suc, fin_zero(Nat.0))) =
        matrix2_entry_00[F](Nat.0, a) * matrix2_entry_01[F](Nat.0, b) + matrix2_entry_01[F](Nat.0, a) * matrix2_entry_11[F](Nat.0, b)
}

/// The lower-left entry of the product of two two-by-two matrices.
theorem matrix2_mul_entry_10[F: Ring](a: Matrix[F, Nat.0.suc.suc, Nat.0.suc.suc], b: Matrix[F, Nat.0.suc.suc, Nat.0.suc.suc]) {
    matrix_entry[F](Nat.0.suc.suc, Nat.0.suc.suc, square_matrix_mul[F](Nat.0.suc.suc, a, b), fin_succ(Nat.0.suc, fin_zero(Nat.0)), fin_zero(Nat.0.suc)) =
        matrix2_entry_10[F](Nat.0, a) * matrix2_entry_00[F](Nat.0, b) + matrix2_entry_11[F](Nat.0, a) * matrix2_entry_10[F](Nat.0, b)
} by {
    matrix_entry_mul_two[F](a, b, fin_succ(Nat.0.suc, fin_zero(Nat.0)), fin_zero(Nat.0.suc))
    matrix_entry[F](Nat.0.suc.suc, Nat.0.suc.suc, square_matrix_mul[F](Nat.0.suc.suc, a, b), fin_succ(Nat.0.suc, fin_zero(Nat.0)), fin_zero(Nat.0.suc)) =
        matrix_entry[F](Nat.0.suc.suc, Nat.0.suc.suc, a, fin_succ(Nat.0.suc, fin_zero(Nat.0)), fin_zero(Nat.0.suc)) *
            matrix_entry[F](Nat.0.suc.suc, Nat.0.suc.suc, b, fin_zero(Nat.0.suc), fin_zero(Nat.0.suc)) +
        matrix_entry[F](Nat.0.suc.suc, Nat.0.suc.suc, a, fin_succ(Nat.0.suc, fin_zero(Nat.0)), fin_succ(Nat.0.suc, fin_zero(Nat.0))) *
            matrix_entry[F](Nat.0.suc.suc, Nat.0.suc.suc, b, fin_succ(Nat.0.suc, fin_zero(Nat.0)), fin_zero(Nat.0.suc))
    matrix_entry[F](Nat.0.suc.suc, Nat.0.suc.suc, square_matrix_mul[F](Nat.0.suc.suc, a, b), fin_succ(Nat.0.suc, fin_zero(Nat.0)), fin_zero(Nat.0.suc)) =
        matrix2_entry_10[F](Nat.0, a) * matrix2_entry_00[F](Nat.0, b) + matrix2_entry_11[F](Nat.0, a) * matrix2_entry_10[F](Nat.0, b)
}

/// The lower-right entry of the product of two two-by-two matrices.
theorem matrix2_mul_entry_11[F: Ring](a: Matrix[F, Nat.0.suc.suc, Nat.0.suc.suc], b: Matrix[F, Nat.0.suc.suc, Nat.0.suc.suc]) {
    matrix_entry[F](Nat.0.suc.suc, Nat.0.suc.suc, square_matrix_mul[F](Nat.0.suc.suc, a, b), fin_succ(Nat.0.suc, fin_zero(Nat.0)), fin_succ(Nat.0.suc, fin_zero(Nat.0))) =
        matrix2_entry_10[F](Nat.0, a) * matrix2_entry_01[F](Nat.0, b) + matrix2_entry_11[F](Nat.0, a) * matrix2_entry_11[F](Nat.0, b)
} by {
    matrix_entry_mul_two[F](a, b, fin_succ(Nat.0.suc, fin_zero(Nat.0)), fin_succ(Nat.0.suc, fin_zero(Nat.0)))
    matrix_entry[F](Nat.0.suc.suc, Nat.0.suc.suc, square_matrix_mul[F](Nat.0.suc.suc, a, b), fin_succ(Nat.0.suc, fin_zero(Nat.0)), fin_succ(Nat.0.suc, fin_zero(Nat.0))) =
        matrix_entry[F](Nat.0.suc.suc, Nat.0.suc.suc, a, fin_succ(Nat.0.suc, fin_zero(Nat.0)), fin_zero(Nat.0.suc)) *
            matrix_entry[F](Nat.0.suc.suc, Nat.0.suc.suc, b, fin_zero(Nat.0.suc), fin_succ(Nat.0.suc, fin_zero(Nat.0))) +
        matrix_entry[F](Nat.0.suc.suc, Nat.0.suc.suc, a, fin_succ(Nat.0.suc, fin_zero(Nat.0)), fin_succ(Nat.0.suc, fin_zero(Nat.0))) *
            matrix_entry[F](Nat.0.suc.suc, Nat.0.suc.suc, b, fin_succ(Nat.0.suc, fin_zero(Nat.0)), fin_succ(Nat.0.suc, fin_zero(Nat.0)))
    matrix_entry[F](Nat.0.suc.suc, Nat.0.suc.suc, square_matrix_mul[F](Nat.0.suc.suc, a, b), fin_succ(Nat.0.suc, fin_zero(Nat.0)), fin_succ(Nat.0.suc, fin_zero(Nat.0))) =
        matrix2_entry_10[F](Nat.0, a) * matrix2_entry_01[F](Nat.0, b) + matrix2_entry_11[F](Nat.0, a) * matrix2_entry_11[F](Nat.0, b)
}
// ---------------------------------------------------------------------------
// The adjugate identity: A * adj(A) = det(A) * I
// ---------------------------------------------------------------------------

/// The upper-left entry of `A * adj(A)` is the determinant of `A`.
theorem matrix2_adjugate_mul_entry_00[F: Field](a: Matrix[F, Nat.0.suc.suc, Nat.0.suc.suc]) {
    matrix_entry[F](Nat.0.suc.suc, Nat.0.suc.suc, square_matrix_mul[F](Nat.0.suc.suc, a, matrix2_adjugate[F](Nat.0, a)), fin_zero(Nat.0.suc), fin_zero(Nat.0.suc)) =
        matrix2_det[F](Nat.0, a)
} by {
    matrix2_mul_entry_00[F](a, matrix2_adjugate[F](Nat.0, a))
    matrix_entry[F](Nat.0.suc.suc, Nat.0.suc.suc, square_matrix_mul[F](Nat.0.suc.suc, a, matrix2_adjugate[F](Nat.0, a)), fin_zero(Nat.0.suc), fin_zero(Nat.0.suc)) =
        matrix2_entry_00[F](Nat.0, a) * matrix2_entry_00[F](Nat.0, matrix2_adjugate[F](Nat.0, a)) +
        matrix2_entry_01[F](Nat.0, a) * matrix2_entry_10[F](Nat.0, matrix2_adjugate[F](Nat.0, a))
    matrix2_adjugate_matrix_entry[F](a, fin_zero(Nat.0.suc), fin_zero(Nat.0.suc))
    matrix2_adjugate_matrix_entry[F](a, fin_succ(Nat.0.suc, fin_zero(Nat.0)), fin_zero(Nat.0.suc))
    matrix2_entry_00[F](Nat.0, matrix2_adjugate[F](Nat.0, a)) =
        matrix2_adjugate_entry[F](Nat.0, a, fin_zero(Nat.0.suc), fin_zero(Nat.0.suc))
    matrix2_entry_10[F](Nat.0, matrix2_adjugate[F](Nat.0, a)) =
        matrix2_adjugate_entry[F](Nat.0, a, fin_succ(Nat.0.suc, fin_zero(Nat.0)), fin_zero(Nat.0.suc))
    matrix2_adjugate_entry_00[F](a)
    matrix2_adjugate_entry_10[F](a)
    matrix_entry[F](Nat.0.suc.suc, Nat.0.suc.suc, square_matrix_mul[F](Nat.0.suc.suc, a, matrix2_adjugate[F](Nat.0, a)), fin_zero(Nat.0.suc), fin_zero(Nat.0.suc)) =
        matrix2_entry_00[F](Nat.0, a) * matrix2_entry_11[F](Nat.0, a) +
        matrix2_entry_01[F](Nat.0, a) * (-matrix2_entry_10[F](Nat.0, a))
    matrix2_entry_01[F](Nat.0, a) * (-matrix2_entry_10[F](Nat.0, a)) =
        -(matrix2_entry_01[F](Nat.0, a) * matrix2_entry_10[F](Nat.0, a))
    matrix_entry[F](Nat.0.suc.suc, Nat.0.suc.suc, square_matrix_mul[F](Nat.0.suc.suc, a, matrix2_adjugate[F](Nat.0, a)), fin_zero(Nat.0.suc), fin_zero(Nat.0.suc)) =
        matrix2_entry_00[F](Nat.0, a) * matrix2_entry_11[F](Nat.0, a) -
        matrix2_entry_01[F](Nat.0, a) * matrix2_entry_10[F](Nat.0, a)
    matrix2_det_unfold[F](a)
    matrix_entry[F](Nat.0.suc.suc, Nat.0.suc.suc, square_matrix_mul[F](Nat.0.suc.suc, a, matrix2_adjugate[F](Nat.0, a)), fin_zero(Nat.0.suc), fin_zero(Nat.0.suc)) =
        matrix2_det[F](Nat.0, a)
}

/// The upper-right entry of `A * adj(A)` is zero.
theorem matrix2_adjugate_mul_entry_01[F: Field](a: Matrix[F, Nat.0.suc.suc, Nat.0.suc.suc]) {
    matrix_entry[F](Nat.0.suc.suc, Nat.0.suc.suc, square_matrix_mul[F](Nat.0.suc.suc, a, matrix2_adjugate[F](Nat.0, a)), fin_zero(Nat.0.suc), fin_succ(Nat.0.suc, fin_zero(Nat.0))) =
        F.0
} by {
    matrix2_mul_entry_01[F](a, matrix2_adjugate[F](Nat.0, a))
    matrix_entry[F](Nat.0.suc.suc, Nat.0.suc.suc, square_matrix_mul[F](Nat.0.suc.suc, a, matrix2_adjugate[F](Nat.0, a)), fin_zero(Nat.0.suc), fin_succ(Nat.0.suc, fin_zero(Nat.0))) =
        matrix2_entry_00[F](Nat.0, a) * matrix2_entry_01[F](Nat.0, matrix2_adjugate[F](Nat.0, a)) +
        matrix2_entry_01[F](Nat.0, a) * matrix2_entry_11[F](Nat.0, matrix2_adjugate[F](Nat.0, a))
    matrix2_adjugate_matrix_entry[F](a, fin_zero(Nat.0.suc), fin_succ(Nat.0.suc, fin_zero(Nat.0)))
    matrix2_adjugate_matrix_entry[F](a, fin_succ(Nat.0.suc, fin_zero(Nat.0)), fin_succ(Nat.0.suc, fin_zero(Nat.0)))
    matrix2_entry_01[F](Nat.0, matrix2_adjugate[F](Nat.0, a)) =
        matrix2_adjugate_entry[F](Nat.0, a, fin_zero(Nat.0.suc), fin_succ(Nat.0.suc, fin_zero(Nat.0)))
    matrix2_entry_11[F](Nat.0, matrix2_adjugate[F](Nat.0, a)) =
        matrix2_adjugate_entry[F](Nat.0, a, fin_succ(Nat.0.suc, fin_zero(Nat.0)), fin_succ(Nat.0.suc, fin_zero(Nat.0)))
    matrix2_adjugate_entry_01[F](a)
    matrix2_adjugate_entry_11[F](a)
    matrix_entry[F](Nat.0.suc.suc, Nat.0.suc.suc, square_matrix_mul[F](Nat.0.suc.suc, a, matrix2_adjugate[F](Nat.0, a)), fin_zero(Nat.0.suc), fin_succ(Nat.0.suc, fin_zero(Nat.0))) =
        matrix2_entry_00[F](Nat.0, a) * (-matrix2_entry_01[F](Nat.0, a)) +
        matrix2_entry_01[F](Nat.0, a) * matrix2_entry_00[F](Nat.0, a)
    matrix2_entry_00[F](Nat.0, a) * (-matrix2_entry_01[F](Nat.0, a)) =
        -(matrix2_entry_00[F](Nat.0, a) * matrix2_entry_01[F](Nat.0, a))
    matrix2_entry_01[F](Nat.0, a) * matrix2_entry_00[F](Nat.0, a) =
        matrix2_entry_00[F](Nat.0, a) * matrix2_entry_01[F](Nat.0, a)
    -(matrix2_entry_00[F](Nat.0, a) * matrix2_entry_01[F](Nat.0, a)) +
        matrix2_entry_00[F](Nat.0, a) * matrix2_entry_01[F](Nat.0, a) = F.0
    matrix_entry[F](Nat.0.suc.suc, Nat.0.suc.suc, square_matrix_mul[F](Nat.0.suc.suc, a, matrix2_adjugate[F](Nat.0, a)), fin_zero(Nat.0.suc), fin_succ(Nat.0.suc, fin_zero(Nat.0))) =
        F.0
}

/// The lower-left entry of `A * adj(A)` is zero.
theorem matrix2_adjugate_mul_entry_10[F: Field](a: Matrix[F, Nat.0.suc.suc, Nat.0.suc.suc]) {
    matrix_entry[F](Nat.0.suc.suc, Nat.0.suc.suc, square_matrix_mul[F](Nat.0.suc.suc, a, matrix2_adjugate[F](Nat.0, a)), fin_succ(Nat.0.suc, fin_zero(Nat.0)), fin_zero(Nat.0.suc)) =
        F.0
} by {
    matrix2_mul_entry_10[F](a, matrix2_adjugate[F](Nat.0, a))
    matrix_entry[F](Nat.0.suc.suc, Nat.0.suc.suc, square_matrix_mul[F](Nat.0.suc.suc, a, matrix2_adjugate[F](Nat.0, a)), fin_succ(Nat.0.suc, fin_zero(Nat.0)), fin_zero(Nat.0.suc)) =
        matrix2_entry_10[F](Nat.0, a) * matrix2_entry_00[F](Nat.0, matrix2_adjugate[F](Nat.0, a)) +
        matrix2_entry_11[F](Nat.0, a) * matrix2_entry_10[F](Nat.0, matrix2_adjugate[F](Nat.0, a))
    matrix2_adjugate_matrix_entry[F](a, fin_zero(Nat.0.suc), fin_zero(Nat.0.suc))
    matrix2_adjugate_matrix_entry[F](a, fin_succ(Nat.0.suc, fin_zero(Nat.0)), fin_zero(Nat.0.suc))
    matrix2_entry_00[F](Nat.0, matrix2_adjugate[F](Nat.0, a)) =
        matrix2_adjugate_entry[F](Nat.0, a, fin_zero(Nat.0.suc), fin_zero(Nat.0.suc))
    matrix2_entry_10[F](Nat.0, matrix2_adjugate[F](Nat.0, a)) =
        matrix2_adjugate_entry[F](Nat.0, a, fin_succ(Nat.0.suc, fin_zero(Nat.0)), fin_zero(Nat.0.suc))
    matrix2_adjugate_entry_00[F](a)
    matrix2_adjugate_entry_10[F](a)
    matrix_entry[F](Nat.0.suc.suc, Nat.0.suc.suc, square_matrix_mul[F](Nat.0.suc.suc, a, matrix2_adjugate[F](Nat.0, a)), fin_succ(Nat.0.suc, fin_zero(Nat.0)), fin_zero(Nat.0.suc)) =
        matrix2_entry_10[F](Nat.0, a) * matrix2_entry_11[F](Nat.0, a) +
        matrix2_entry_11[F](Nat.0, a) * (-matrix2_entry_10[F](Nat.0, a))
    matrix2_entry_11[F](Nat.0, a) * (-matrix2_entry_10[F](Nat.0, a)) =
        -(matrix2_entry_11[F](Nat.0, a) * matrix2_entry_10[F](Nat.0, a))
    matrix2_entry_11[F](Nat.0, a) * matrix2_entry_10[F](Nat.0, a) =
        matrix2_entry_10[F](Nat.0, a) * matrix2_entry_11[F](Nat.0, a)
    matrix2_entry_10[F](Nat.0, a) * matrix2_entry_11[F](Nat.0, a) +
        -(matrix2_entry_10[F](Nat.0, a) * matrix2_entry_11[F](Nat.0, a)) = F.0
    matrix_entry[F](Nat.0.suc.suc, Nat.0.suc.suc, square_matrix_mul[F](Nat.0.suc.suc, a, matrix2_adjugate[F](Nat.0, a)), fin_succ(Nat.0.suc, fin_zero(Nat.0)), fin_zero(Nat.0.suc)) =
        F.0
}

/// The lower-right entry of `A * adj(A)` is the determinant of `A`.
theorem matrix2_adjugate_mul_entry_11[F: Field](a: Matrix[F, Nat.0.suc.suc, Nat.0.suc.suc]) {
    matrix_entry[F](Nat.0.suc.suc, Nat.0.suc.suc, square_matrix_mul[F](Nat.0.suc.suc, a, matrix2_adjugate[F](Nat.0, a)), fin_succ(Nat.0.suc, fin_zero(Nat.0)), fin_succ(Nat.0.suc, fin_zero(Nat.0))) =
        matrix2_det[F](Nat.0, a)
} by {
    matrix2_mul_entry_11[F](a, matrix2_adjugate[F](Nat.0, a))
    matrix_entry[F](Nat.0.suc.suc, Nat.0.suc.suc, square_matrix_mul[F](Nat.0.suc.suc, a, matrix2_adjugate[F](Nat.0, a)), fin_succ(Nat.0.suc, fin_zero(Nat.0)), fin_succ(Nat.0.suc, fin_zero(Nat.0))) =
        matrix2_entry_10[F](Nat.0, a) * matrix2_entry_01[F](Nat.0, matrix2_adjugate[F](Nat.0, a)) +
        matrix2_entry_11[F](Nat.0, a) * matrix2_entry_11[F](Nat.0, matrix2_adjugate[F](Nat.0, a))
    matrix2_adjugate_matrix_entry[F](a, fin_zero(Nat.0.suc), fin_succ(Nat.0.suc, fin_zero(Nat.0)))
    matrix2_adjugate_matrix_entry[F](a, fin_succ(Nat.0.suc, fin_zero(Nat.0)), fin_succ(Nat.0.suc, fin_zero(Nat.0)))
    matrix2_entry_01[F](Nat.0, matrix2_adjugate[F](Nat.0, a)) =
        matrix2_adjugate_entry[F](Nat.0, a, fin_zero(Nat.0.suc), fin_succ(Nat.0.suc, fin_zero(Nat.0)))
    matrix2_entry_11[F](Nat.0, matrix2_adjugate[F](Nat.0, a)) =
        matrix2_adjugate_entry[F](Nat.0, a, fin_succ(Nat.0.suc, fin_zero(Nat.0)), fin_succ(Nat.0.suc, fin_zero(Nat.0)))
    matrix2_adjugate_entry_01[F](a)
    matrix2_adjugate_entry_11[F](a)
    matrix_entry[F](Nat.0.suc.suc, Nat.0.suc.suc, square_matrix_mul[F](Nat.0.suc.suc, a, matrix2_adjugate[F](Nat.0, a)), fin_succ(Nat.0.suc, fin_zero(Nat.0)), fin_succ(Nat.0.suc, fin_zero(Nat.0))) =
        matrix2_entry_10[F](Nat.0, a) * (-matrix2_entry_01[F](Nat.0, a)) +
        matrix2_entry_11[F](Nat.0, a) * matrix2_entry_00[F](Nat.0, a)
    matrix2_entry_10[F](Nat.0, a) * (-matrix2_entry_01[F](Nat.0, a)) =
        -(matrix2_entry_10[F](Nat.0, a) * matrix2_entry_01[F](Nat.0, a))
    matrix2_entry_10[F](Nat.0, a) * matrix2_entry_01[F](Nat.0, a) =
        matrix2_entry_01[F](Nat.0, a) * matrix2_entry_10[F](Nat.0, a)
    -(matrix2_entry_01[F](Nat.0, a) * matrix2_entry_10[F](Nat.0, a)) +
        matrix2_entry_11[F](Nat.0, a) * matrix2_entry_00[F](Nat.0, a) =
        matrix2_entry_00[F](Nat.0, a) * matrix2_entry_11[F](Nat.0, a) -
        matrix2_entry_01[F](Nat.0, a) * matrix2_entry_10[F](Nat.0, a)
    matrix2_det_unfold[F](a)
    matrix_entry[F](Nat.0.suc.suc, Nat.0.suc.suc, square_matrix_mul[F](Nat.0.suc.suc, a, matrix2_adjugate[F](Nat.0, a)), fin_succ(Nat.0.suc, fin_zero(Nat.0)), fin_succ(Nat.0.suc, fin_zero(Nat.0))) =
        matrix2_det[F](Nat.0, a)
}

/// An entry of `A * adj(A)` is the corresponding entry of the determinant
/// times the identity: `det(A)` on the diagonal and zero off it.
///
/// The matrix-level statement `A * adj(A) = matrix2_scalar_one(det(A))` has
/// the same content; it is left entrywise because the certificate serializer
/// currently cannot round-trip the unfolding of the scalar-identity matrix
/// (see the note in `algebra/matrix_algebra.ac`).
theorem matrix2_adjugate_identity_entry[F: Field](a: Matrix[F, Nat.0.suc.suc, Nat.0.suc.suc], i: Fin[Nat.0.suc.suc], j: Fin[Nat.0.suc.suc]) {
    matrix_entry[F](Nat.0.suc.suc, Nat.0.suc.suc, square_matrix_mul[F](Nat.0.suc.suc, a, matrix2_adjugate[F](Nat.0, a)), i, j) =
        matrix2_scalar_one_entry[F](Nat.0, matrix2_det[F](Nat.0, a), i, j)
} by {
    if i.value = Nat.0 {
        if j.value = Nat.0 {
            fin_two_zero_value_two
            i.value = fin_zero(Nat.0.suc).value
            ext(Nat.0.suc.suc, i, fin_zero(Nat.0.suc))
            i = fin_zero(Nat.0.suc)
            j.value = fin_zero(Nat.0.suc).value
            ext(Nat.0.suc.suc, j, fin_zero(Nat.0.suc))
            j = fin_zero(Nat.0.suc)
            matrix2_adjugate_mul_entry_00[F](a)
            matrix_entry[F](Nat.0.suc.suc, Nat.0.suc.suc, square_matrix_mul[F](Nat.0.suc.suc, a, matrix2_adjugate[F](Nat.0, a)), i, j) =
                matrix2_det[F](Nat.0, a)
            matrix2_scalar_one_entry[F](Nat.0, matrix2_det[F](Nat.0, a), i, j) =
                matrix2_det[F](Nat.0, a)
            matrix_entry[F](Nat.0.suc.suc, Nat.0.suc.suc, square_matrix_mul[F](Nat.0.suc.suc, a, matrix2_adjugate[F](Nat.0, a)), i, j) =
                matrix2_scalar_one_entry[F](Nat.0, matrix2_det[F](Nat.0, a), i, j)
        } else {
            fin_two_zero_value_two
            i.value = fin_zero(Nat.0.suc).value
            ext(Nat.0.suc.suc, i, fin_zero(Nat.0.suc))
            i = fin_zero(Nat.0.suc)
            fin_two_one_value_two
            j.value = fin_succ(Nat.0.suc, fin_zero(Nat.0)).value
            ext(Nat.0.suc.suc, j, fin_succ(Nat.0.suc, fin_zero(Nat.0)))
            j = fin_succ(Nat.0.suc, fin_zero(Nat.0))
            matrix2_adjugate_mul_entry_01[F](a)
            matrix_entry[F](Nat.0.suc.suc, Nat.0.suc.suc, square_matrix_mul[F](Nat.0.suc.suc, a, matrix2_adjugate[F](Nat.0, a)), i, j) = F.0
            fin_two_zero_ne_one
            matrix2_scalar_one_entry[F](Nat.0, matrix2_det[F](Nat.0, a), i, j) = F.0
            matrix_entry[F](Nat.0.suc.suc, Nat.0.suc.suc, square_matrix_mul[F](Nat.0.suc.suc, a, matrix2_adjugate[F](Nat.0, a)), i, j) =
                matrix2_scalar_one_entry[F](Nat.0, matrix2_det[F](Nat.0, a), i, j)
        }
    } else {
        if j.value = Nat.0 {
            fin_two_one_value_two
            i.value = fin_succ(Nat.0.suc, fin_zero(Nat.0)).value
            ext(Nat.0.suc.suc, i, fin_succ(Nat.0.suc, fin_zero(Nat.0)))
            i = fin_succ(Nat.0.suc, fin_zero(Nat.0))
            fin_two_zero_value_two
            j.value = fin_zero(Nat.0.suc).value
            ext(Nat.0.suc.suc, j, fin_zero(Nat.0.suc))
            j = fin_zero(Nat.0.suc)
            matrix2_adjugate_mul_entry_10[F](a)
            matrix_entry[F](Nat.0.suc.suc, Nat.0.suc.suc, square_matrix_mul[F](Nat.0.suc.suc, a, matrix2_adjugate[F](Nat.0, a)), i, j) = F.0
            fin_two_zero_ne_one
            matrix2_scalar_one_entry[F](Nat.0, matrix2_det[F](Nat.0, a), i, j) = F.0
            matrix_entry[F](Nat.0.suc.suc, Nat.0.suc.suc, square_matrix_mul[F](Nat.0.suc.suc, a, matrix2_adjugate[F](Nat.0, a)), i, j) =
                matrix2_scalar_one_entry[F](Nat.0, matrix2_det[F](Nat.0, a), i, j)
        } else {
            fin_two_one_value_two
            i.value = fin_succ(Nat.0.suc, fin_zero(Nat.0)).value
            ext(Nat.0.suc.suc, i, fin_succ(Nat.0.suc, fin_zero(Nat.0)))
            i = fin_succ(Nat.0.suc, fin_zero(Nat.0))
            j.value = fin_succ(Nat.0.suc, fin_zero(Nat.0)).value
            ext(Nat.0.suc.suc, j, fin_succ(Nat.0.suc, fin_zero(Nat.0)))
            j = fin_succ(Nat.0.suc, fin_zero(Nat.0))
            matrix2_adjugate_mul_entry_11[F](a)
            matrix_entry[F](Nat.0.suc.suc, Nat.0.suc.suc, square_matrix_mul[F](Nat.0.suc.suc, a, matrix2_adjugate[F](Nat.0, a)), i, j) =
                matrix2_det[F](Nat.0, a)
            matrix2_scalar_one_entry[F](Nat.0, matrix2_det[F](Nat.0, a), i, j) =
                matrix2_det[F](Nat.0, a)
            matrix_entry[F](Nat.0.suc.suc, Nat.0.suc.suc, square_matrix_mul[F](Nat.0.suc.suc, a, matrix2_adjugate[F](Nat.0, a)), i, j) =
                matrix2_scalar_one_entry[F](Nat.0, matrix2_det[F](Nat.0, a), i, j)
        }
    }
}

// ---------------------------------------------------------------------------
// The left adjugate identity: adj(A) * A = det(A) * I
// ---------------------------------------------------------------------------

/// The upper-left entry of `adj(A) * A` is the determinant of `A`.
theorem matrix2_adjugate_mul_left_entry_00[F: Field](a: Matrix[F, Nat.0.suc.suc, Nat.0.suc.suc]) {
    matrix_entry[F](Nat.0.suc.suc, Nat.0.suc.suc, square_matrix_mul[F](Nat.0.suc.suc, matrix2_adjugate[F](Nat.0, a), a), fin_zero(Nat.0.suc), fin_zero(Nat.0.suc)) =
        matrix2_det[F](Nat.0, a)
} by {
    matrix2_mul_entry_00[F](matrix2_adjugate[F](Nat.0, a), a)
    matrix_entry[F](Nat.0.suc.suc, Nat.0.suc.suc, square_matrix_mul[F](Nat.0.suc.suc, matrix2_adjugate[F](Nat.0, a), a), fin_zero(Nat.0.suc), fin_zero(Nat.0.suc)) =
        matrix2_entry_00[F](Nat.0, matrix2_adjugate[F](Nat.0, a)) * matrix2_entry_00[F](Nat.0, a) +
        matrix2_entry_01[F](Nat.0, matrix2_adjugate[F](Nat.0, a)) * matrix2_entry_10[F](Nat.0, a)
    matrix2_adjugate_matrix_entry[F](a, fin_zero(Nat.0.suc), fin_zero(Nat.0.suc))
    matrix2_adjugate_matrix_entry[F](a, fin_zero(Nat.0.suc), fin_succ(Nat.0.suc, fin_zero(Nat.0)))
    matrix2_entry_00[F](Nat.0, matrix2_adjugate[F](Nat.0, a)) =
        matrix2_adjugate_entry[F](Nat.0, a, fin_zero(Nat.0.suc), fin_zero(Nat.0.suc))
    matrix2_entry_01[F](Nat.0, matrix2_adjugate[F](Nat.0, a)) =
        matrix2_adjugate_entry[F](Nat.0, a, fin_zero(Nat.0.suc), fin_succ(Nat.0.suc, fin_zero(Nat.0)))
    matrix2_adjugate_entry_00[F](a)
    matrix2_adjugate_entry_01[F](a)
    matrix_entry[F](Nat.0.suc.suc, Nat.0.suc.suc, square_matrix_mul[F](Nat.0.suc.suc, matrix2_adjugate[F](Nat.0, a), a), fin_zero(Nat.0.suc), fin_zero(Nat.0.suc)) =
        matrix2_entry_11[F](Nat.0, a) * matrix2_entry_00[F](Nat.0, a) +
        (-matrix2_entry_01[F](Nat.0, a)) * matrix2_entry_10[F](Nat.0, a)
    (-matrix2_entry_01[F](Nat.0, a)) * matrix2_entry_10[F](Nat.0, a) =
        -(matrix2_entry_01[F](Nat.0, a) * matrix2_entry_10[F](Nat.0, a))
    matrix2_entry_11[F](Nat.0, a) * matrix2_entry_00[F](Nat.0, a) =
        matrix2_entry_00[F](Nat.0, a) * matrix2_entry_11[F](Nat.0, a)
    matrix_entry[F](Nat.0.suc.suc, Nat.0.suc.suc, square_matrix_mul[F](Nat.0.suc.suc, matrix2_adjugate[F](Nat.0, a), a), fin_zero(Nat.0.suc), fin_zero(Nat.0.suc)) =
        matrix2_entry_00[F](Nat.0, a) * matrix2_entry_11[F](Nat.0, a) -
        matrix2_entry_01[F](Nat.0, a) * matrix2_entry_10[F](Nat.0, a)
    matrix2_det_unfold[F](a)
    matrix_entry[F](Nat.0.suc.suc, Nat.0.suc.suc, square_matrix_mul[F](Nat.0.suc.suc, matrix2_adjugate[F](Nat.0, a), a), fin_zero(Nat.0.suc), fin_zero(Nat.0.suc)) =
        matrix2_det[F](Nat.0, a)
}

/// The upper-right entry of `adj(A) * A` is zero.
theorem matrix2_adjugate_mul_left_entry_01[F: Field](a: Matrix[F, Nat.0.suc.suc, Nat.0.suc.suc]) {
    matrix_entry[F](Nat.0.suc.suc, Nat.0.suc.suc, square_matrix_mul[F](Nat.0.suc.suc, matrix2_adjugate[F](Nat.0, a), a), fin_zero(Nat.0.suc), fin_succ(Nat.0.suc, fin_zero(Nat.0))) =
        F.0
} by {
    matrix2_mul_entry_01[F](matrix2_adjugate[F](Nat.0, a), a)
    matrix_entry[F](Nat.0.suc.suc, Nat.0.suc.suc, square_matrix_mul[F](Nat.0.suc.suc, matrix2_adjugate[F](Nat.0, a), a), fin_zero(Nat.0.suc), fin_succ(Nat.0.suc, fin_zero(Nat.0))) =
        matrix2_entry_00[F](Nat.0, matrix2_adjugate[F](Nat.0, a)) * matrix2_entry_01[F](Nat.0, a) +
        matrix2_entry_01[F](Nat.0, matrix2_adjugate[F](Nat.0, a)) * matrix2_entry_11[F](Nat.0, a)
    matrix2_adjugate_matrix_entry[F](a, fin_zero(Nat.0.suc), fin_zero(Nat.0.suc))
    matrix2_adjugate_matrix_entry[F](a, fin_zero(Nat.0.suc), fin_succ(Nat.0.suc, fin_zero(Nat.0)))
    matrix2_entry_00[F](Nat.0, matrix2_adjugate[F](Nat.0, a)) =
        matrix2_adjugate_entry[F](Nat.0, a, fin_zero(Nat.0.suc), fin_zero(Nat.0.suc))
    matrix2_entry_01[F](Nat.0, matrix2_adjugate[F](Nat.0, a)) =
        matrix2_adjugate_entry[F](Nat.0, a, fin_zero(Nat.0.suc), fin_succ(Nat.0.suc, fin_zero(Nat.0)))
    matrix2_adjugate_entry_00[F](a)
    matrix2_adjugate_entry_01[F](a)
    matrix_entry[F](Nat.0.suc.suc, Nat.0.suc.suc, square_matrix_mul[F](Nat.0.suc.suc, matrix2_adjugate[F](Nat.0, a), a), fin_zero(Nat.0.suc), fin_succ(Nat.0.suc, fin_zero(Nat.0))) =
        matrix2_entry_11[F](Nat.0, a) * matrix2_entry_01[F](Nat.0, a) +
        (-matrix2_entry_01[F](Nat.0, a)) * matrix2_entry_11[F](Nat.0, a)
    (-matrix2_entry_01[F](Nat.0, a)) * matrix2_entry_11[F](Nat.0, a) =
        -(matrix2_entry_01[F](Nat.0, a) * matrix2_entry_11[F](Nat.0, a))
    matrix2_entry_11[F](Nat.0, a) * matrix2_entry_01[F](Nat.0, a) =
        matrix2_entry_01[F](Nat.0, a) * matrix2_entry_11[F](Nat.0, a)
    matrix2_entry_01[F](Nat.0, a) * matrix2_entry_11[F](Nat.0, a) +
        -(matrix2_entry_01[F](Nat.0, a) * matrix2_entry_11[F](Nat.0, a)) = F.0
    matrix_entry[F](Nat.0.suc.suc, Nat.0.suc.suc, square_matrix_mul[F](Nat.0.suc.suc, matrix2_adjugate[F](Nat.0, a), a), fin_zero(Nat.0.suc), fin_succ(Nat.0.suc, fin_zero(Nat.0))) =
        F.0
}

/// The lower-left entry of `adj(A) * A` is zero.
theorem matrix2_adjugate_mul_left_entry_10[F: Field](a: Matrix[F, Nat.0.suc.suc, Nat.0.suc.suc]) {
    matrix_entry[F](Nat.0.suc.suc, Nat.0.suc.suc, square_matrix_mul[F](Nat.0.suc.suc, matrix2_adjugate[F](Nat.0, a), a), fin_succ(Nat.0.suc, fin_zero(Nat.0)), fin_zero(Nat.0.suc)) =
        F.0
} by {
    matrix2_mul_entry_10[F](matrix2_adjugate[F](Nat.0, a), a)
    matrix_entry[F](Nat.0.suc.suc, Nat.0.suc.suc, square_matrix_mul[F](Nat.0.suc.suc, matrix2_adjugate[F](Nat.0, a), a), fin_succ(Nat.0.suc, fin_zero(Nat.0)), fin_zero(Nat.0.suc)) =
        matrix2_entry_10[F](Nat.0, matrix2_adjugate[F](Nat.0, a)) * matrix2_entry_00[F](Nat.0, a) +
        matrix2_entry_11[F](Nat.0, matrix2_adjugate[F](Nat.0, a)) * matrix2_entry_10[F](Nat.0, a)
    matrix2_adjugate_matrix_entry[F](a, fin_succ(Nat.0.suc, fin_zero(Nat.0)), fin_zero(Nat.0.suc))
    matrix2_adjugate_matrix_entry[F](a, fin_succ(Nat.0.suc, fin_zero(Nat.0)), fin_succ(Nat.0.suc, fin_zero(Nat.0)))
    matrix2_entry_10[F](Nat.0, matrix2_adjugate[F](Nat.0, a)) =
        matrix2_adjugate_entry[F](Nat.0, a, fin_succ(Nat.0.suc, fin_zero(Nat.0)), fin_zero(Nat.0.suc))
    matrix2_entry_11[F](Nat.0, matrix2_adjugate[F](Nat.0, a)) =
        matrix2_adjugate_entry[F](Nat.0, a, fin_succ(Nat.0.suc, fin_zero(Nat.0)), fin_succ(Nat.0.suc, fin_zero(Nat.0)))
    matrix2_adjugate_entry_10[F](a)
    matrix2_adjugate_entry_11[F](a)
    matrix_entry[F](Nat.0.suc.suc, Nat.0.suc.suc, square_matrix_mul[F](Nat.0.suc.suc, matrix2_adjugate[F](Nat.0, a), a), fin_succ(Nat.0.suc, fin_zero(Nat.0)), fin_zero(Nat.0.suc)) =
        (-matrix2_entry_10[F](Nat.0, a)) * matrix2_entry_00[F](Nat.0, a) +
        matrix2_entry_00[F](Nat.0, a) * matrix2_entry_10[F](Nat.0, a)
    (-matrix2_entry_10[F](Nat.0, a)) * matrix2_entry_00[F](Nat.0, a) =
        -(matrix2_entry_10[F](Nat.0, a) * matrix2_entry_00[F](Nat.0, a))
    matrix2_entry_10[F](Nat.0, a) * matrix2_entry_00[F](Nat.0, a) =
        matrix2_entry_00[F](Nat.0, a) * matrix2_entry_10[F](Nat.0, a)
    -(matrix2_entry_00[F](Nat.0, a) * matrix2_entry_10[F](Nat.0, a)) +
        matrix2_entry_00[F](Nat.0, a) * matrix2_entry_10[F](Nat.0, a) = F.0
    matrix_entry[F](Nat.0.suc.suc, Nat.0.suc.suc, square_matrix_mul[F](Nat.0.suc.suc, matrix2_adjugate[F](Nat.0, a), a), fin_succ(Nat.0.suc, fin_zero(Nat.0)), fin_zero(Nat.0.suc)) =
        F.0
}

/// The lower-right entry of `adj(A) * A` is the determinant of `A`.
theorem matrix2_adjugate_mul_left_entry_11[F: Field](a: Matrix[F, Nat.0.suc.suc, Nat.0.suc.suc]) {
    matrix_entry[F](Nat.0.suc.suc, Nat.0.suc.suc, square_matrix_mul[F](Nat.0.suc.suc, matrix2_adjugate[F](Nat.0, a), a), fin_succ(Nat.0.suc, fin_zero(Nat.0)), fin_succ(Nat.0.suc, fin_zero(Nat.0))) =
        matrix2_det[F](Nat.0, a)
} by {
    matrix2_mul_entry_11[F](matrix2_adjugate[F](Nat.0, a), a)
    matrix_entry[F](Nat.0.suc.suc, Nat.0.suc.suc, square_matrix_mul[F](Nat.0.suc.suc, matrix2_adjugate[F](Nat.0, a), a), fin_succ(Nat.0.suc, fin_zero(Nat.0)), fin_succ(Nat.0.suc, fin_zero(Nat.0))) =
        matrix2_entry_10[F](Nat.0, matrix2_adjugate[F](Nat.0, a)) * matrix2_entry_01[F](Nat.0, a) +
        matrix2_entry_11[F](Nat.0, matrix2_adjugate[F](Nat.0, a)) * matrix2_entry_11[F](Nat.0, a)
    matrix2_adjugate_matrix_entry[F](a, fin_succ(Nat.0.suc, fin_zero(Nat.0)), fin_zero(Nat.0.suc))
    matrix2_adjugate_matrix_entry[F](a, fin_succ(Nat.0.suc, fin_zero(Nat.0)), fin_succ(Nat.0.suc, fin_zero(Nat.0)))
    matrix2_entry_10[F](Nat.0, matrix2_adjugate[F](Nat.0, a)) =
        matrix2_adjugate_entry[F](Nat.0, a, fin_succ(Nat.0.suc, fin_zero(Nat.0)), fin_zero(Nat.0.suc))
    matrix2_entry_11[F](Nat.0, matrix2_adjugate[F](Nat.0, a)) =
        matrix2_adjugate_entry[F](Nat.0, a, fin_succ(Nat.0.suc, fin_zero(Nat.0)), fin_succ(Nat.0.suc, fin_zero(Nat.0)))
    matrix2_adjugate_entry_10[F](a)
    matrix2_adjugate_entry_11[F](a)
    matrix_entry[F](Nat.0.suc.suc, Nat.0.suc.suc, square_matrix_mul[F](Nat.0.suc.suc, matrix2_adjugate[F](Nat.0, a), a), fin_succ(Nat.0.suc, fin_zero(Nat.0)), fin_succ(Nat.0.suc, fin_zero(Nat.0))) =
        (-matrix2_entry_10[F](Nat.0, a)) * matrix2_entry_01[F](Nat.0, a) +
        matrix2_entry_00[F](Nat.0, a) * matrix2_entry_11[F](Nat.0, a)
    (-matrix2_entry_10[F](Nat.0, a)) * matrix2_entry_01[F](Nat.0, a) =
        -(matrix2_entry_10[F](Nat.0, a) * matrix2_entry_01[F](Nat.0, a))
    matrix2_entry_10[F](Nat.0, a) * matrix2_entry_01[F](Nat.0, a) =
        matrix2_entry_01[F](Nat.0, a) * matrix2_entry_10[F](Nat.0, a)
    -(matrix2_entry_01[F](Nat.0, a) * matrix2_entry_10[F](Nat.0, a)) +
        matrix2_entry_00[F](Nat.0, a) * matrix2_entry_11[F](Nat.0, a) =
        matrix2_entry_00[F](Nat.0, a) * matrix2_entry_11[F](Nat.0, a) -
        matrix2_entry_01[F](Nat.0, a) * matrix2_entry_10[F](Nat.0, a)
    matrix2_det_unfold[F](a)
    matrix_entry[F](Nat.0.suc.suc, Nat.0.suc.suc, square_matrix_mul[F](Nat.0.suc.suc, matrix2_adjugate[F](Nat.0, a), a), fin_succ(Nat.0.suc, fin_zero(Nat.0)), fin_succ(Nat.0.suc, fin_zero(Nat.0))) =
        matrix2_det[F](Nat.0, a)
}

/// An entry of `adj(A) * A` is the corresponding entry of the determinant
/// times the identity.
theorem matrix2_adjugate_identity_left_entry[F: Field](a: Matrix[F, Nat.0.suc.suc, Nat.0.suc.suc], i: Fin[Nat.0.suc.suc], j: Fin[Nat.0.suc.suc]) {
    matrix_entry[F](Nat.0.suc.suc, Nat.0.suc.suc, square_matrix_mul[F](Nat.0.suc.suc, matrix2_adjugate[F](Nat.0, a), a), i, j) =
        matrix2_scalar_one_entry[F](Nat.0, matrix2_det[F](Nat.0, a), i, j)
} by {
    if i.value = Nat.0 {
        if j.value = Nat.0 {
            fin_two_zero_value_two
            i.value = fin_zero(Nat.0.suc).value
            ext(Nat.0.suc.suc, i, fin_zero(Nat.0.suc))
            i = fin_zero(Nat.0.suc)
            j.value = fin_zero(Nat.0.suc).value
            ext(Nat.0.suc.suc, j, fin_zero(Nat.0.suc))
            j = fin_zero(Nat.0.suc)
            matrix2_adjugate_mul_left_entry_00[F](a)
            matrix_entry[F](Nat.0.suc.suc, Nat.0.suc.suc, square_matrix_mul[F](Nat.0.suc.suc, matrix2_adjugate[F](Nat.0, a), a), i, j) =
                matrix2_det[F](Nat.0, a)
            matrix2_scalar_one_entry[F](Nat.0, matrix2_det[F](Nat.0, a), i, j) =
                matrix2_det[F](Nat.0, a)
            matrix_entry[F](Nat.0.suc.suc, Nat.0.suc.suc, square_matrix_mul[F](Nat.0.suc.suc, matrix2_adjugate[F](Nat.0, a), a), i, j) =
                matrix2_scalar_one_entry[F](Nat.0, matrix2_det[F](Nat.0, a), i, j)
        } else {
            fin_two_zero_value_two
            i.value = fin_zero(Nat.0.suc).value
            ext(Nat.0.suc.suc, i, fin_zero(Nat.0.suc))
            i = fin_zero(Nat.0.suc)
            fin_two_one_value_two
            j.value = fin_succ(Nat.0.suc, fin_zero(Nat.0)).value
            ext(Nat.0.suc.suc, j, fin_succ(Nat.0.suc, fin_zero(Nat.0)))
            j = fin_succ(Nat.0.suc, fin_zero(Nat.0))
            matrix2_adjugate_mul_left_entry_01[F](a)
            matrix_entry[F](Nat.0.suc.suc, Nat.0.suc.suc, square_matrix_mul[F](Nat.0.suc.suc, matrix2_adjugate[F](Nat.0, a), a), i, j) = F.0
            fin_two_zero_ne_one
            matrix2_scalar_one_entry[F](Nat.0, matrix2_det[F](Nat.0, a), i, j) = F.0
            matrix_entry[F](Nat.0.suc.suc, Nat.0.suc.suc, square_matrix_mul[F](Nat.0.suc.suc, matrix2_adjugate[F](Nat.0, a), a), i, j) =
                matrix2_scalar_one_entry[F](Nat.0, matrix2_det[F](Nat.0, a), i, j)
        }
    } else {
        if j.value = Nat.0 {
            fin_two_one_value_two
            i.value = fin_succ(Nat.0.suc, fin_zero(Nat.0)).value
            ext(Nat.0.suc.suc, i, fin_succ(Nat.0.suc, fin_zero(Nat.0)))
            i = fin_succ(Nat.0.suc, fin_zero(Nat.0))
            fin_two_zero_value_two
            j.value = fin_zero(Nat.0.suc).value
            ext(Nat.0.suc.suc, j, fin_zero(Nat.0.suc))
            j = fin_zero(Nat.0.suc)
            matrix2_adjugate_mul_left_entry_10[F](a)
            matrix_entry[F](Nat.0.suc.suc, Nat.0.suc.suc, square_matrix_mul[F](Nat.0.suc.suc, matrix2_adjugate[F](Nat.0, a), a), i, j) = F.0
            fin_two_zero_ne_one
            matrix2_scalar_one_entry[F](Nat.0, matrix2_det[F](Nat.0, a), i, j) = F.0
            matrix_entry[F](Nat.0.suc.suc, Nat.0.suc.suc, square_matrix_mul[F](Nat.0.suc.suc, matrix2_adjugate[F](Nat.0, a), a), i, j) =
                matrix2_scalar_one_entry[F](Nat.0, matrix2_det[F](Nat.0, a), i, j)
        } else {
            fin_two_one_value_two
            i.value = fin_succ(Nat.0.suc, fin_zero(Nat.0)).value
            ext(Nat.0.suc.suc, i, fin_succ(Nat.0.suc, fin_zero(Nat.0)))
            i = fin_succ(Nat.0.suc, fin_zero(Nat.0))
            j.value = fin_succ(Nat.0.suc, fin_zero(Nat.0)).value
            ext(Nat.0.suc.suc, j, fin_succ(Nat.0.suc, fin_zero(Nat.0)))
            j = fin_succ(Nat.0.suc, fin_zero(Nat.0))
            matrix2_adjugate_mul_left_entry_11[F](a)
            matrix_entry[F](Nat.0.suc.suc, Nat.0.suc.suc, square_matrix_mul[F](Nat.0.suc.suc, matrix2_adjugate[F](Nat.0, a), a), i, j) =
                matrix2_det[F](Nat.0, a)
            matrix2_scalar_one_entry[F](Nat.0, matrix2_det[F](Nat.0, a), i, j) =
                matrix2_det[F](Nat.0, a)
            matrix_entry[F](Nat.0.suc.suc, Nat.0.suc.suc, square_matrix_mul[F](Nat.0.suc.suc, matrix2_adjugate[F](Nat.0, a), a), i, j) =
                matrix2_scalar_one_entry[F](Nat.0, matrix2_det[F](Nat.0, a), i, j)
        }
    }
}
// ---------------------------------------------------------------------------
// The inverse of a two-by-two matrix
// ---------------------------------------------------------------------------

/// An entry of the inverse matrix is the corresponding inverse entry.
theorem matrix2_inv_matrix_entry[F: Field](
    a: Matrix[F, Nat.0.suc.suc, Nat.0.suc.suc], i: Fin[Nat.0.suc.suc], j: Fin[Nat.0.suc.suc]
) {
    matrix_entry[F](Nat.0.suc.suc, Nat.0.suc.suc, matrix2_inv[F](Nat.0, a), i, j) =
        matrix2_inv_entry[F](Nat.0, a, i, j)
} by {
    matrix_entry[F](Nat.0.suc.suc, Nat.0.suc.suc, matrix2_inv[F](Nat.0, a), i, j) =
        matrix2_inv_entry[F](Nat.0, a, i, j)
}

/// An inverse entry is the determinant inverse times the adjugate entry.
///
/// This is the entrywise content of the formula
/// `inv(A) = (1/det(A)) * adj(A)`; the matrix-level statement needs the
/// scalar-multiple-of-a-matrix helper, whose unfolding the certificate
/// serializer cannot round-trip (see the note in `algebra/matrix_algebra.ac`).
theorem matrix2_inv_entry_unfold[F: Field](
    a: Matrix[F, Nat.0.suc.suc, Nat.0.suc.suc], i: Fin[Nat.0.suc.suc], j: Fin[Nat.0.suc.suc]
) {
    matrix2_inv_entry[F](Nat.0, a, i, j) =
        matrix2_det[F](Nat.0, a).inverse * matrix2_adjugate_entry[F](Nat.0, a, i, j)
} by {
    matrix2_inv_entry[F](Nat.0, a, i, j) =
        matrix2_det[F](Nat.0, a).inverse * matrix2_adjugate_entry[F](Nat.0, a, i, j)
}

/// The upper-left inverse entry is the determinant inverse times the
/// lower-right entry.
theorem matrix2_inv_value_00[F: Field](a: Matrix[F, Nat.0.suc.suc, Nat.0.suc.suc]) {
    matrix2_inv_entry[F](Nat.0, a, fin_zero(Nat.0.suc), fin_zero(Nat.0.suc)) =
        matrix2_det[F](Nat.0, a).inverse * matrix2_entry_11[F](Nat.0, a)
} by {
    matrix2_inv_entry_unfold[F](a, fin_zero(Nat.0.suc), fin_zero(Nat.0.suc))
    matrix2_inv_entry[F](Nat.0, a, fin_zero(Nat.0.suc), fin_zero(Nat.0.suc)) =
        matrix2_det[F](Nat.0, a).inverse * matrix2_adjugate_entry[F](Nat.0, a, fin_zero(Nat.0.suc), fin_zero(Nat.0.suc))
    matrix2_adjugate_entry_00[F](a)
    matrix2_inv_entry[F](Nat.0, a, fin_zero(Nat.0.suc), fin_zero(Nat.0.suc)) =
        matrix2_det[F](Nat.0, a).inverse * matrix2_entry_11[F](Nat.0, a)
}

/// The upper-right inverse entry is the determinant inverse times the negated
/// upper-right entry.
theorem matrix2_inv_value_01[F: Field](a: Matrix[F, Nat.0.suc.suc, Nat.0.suc.suc]) {
    matrix2_inv_entry[F](Nat.0, a, fin_zero(Nat.0.suc), fin_succ(Nat.0.suc, fin_zero(Nat.0))) =
        matrix2_det[F](Nat.0, a).inverse * (-matrix2_entry_01[F](Nat.0, a))
} by {
    matrix2_inv_entry_unfold[F](a, fin_zero(Nat.0.suc), fin_succ(Nat.0.suc, fin_zero(Nat.0)))
    matrix2_inv_entry[F](Nat.0, a, fin_zero(Nat.0.suc), fin_succ(Nat.0.suc, fin_zero(Nat.0))) =
        matrix2_det[F](Nat.0, a).inverse * matrix2_adjugate_entry[F](Nat.0, a, fin_zero(Nat.0.suc), fin_succ(Nat.0.suc, fin_zero(Nat.0)))
    matrix2_adjugate_entry_01[F](a)
    matrix2_inv_entry[F](Nat.0, a, fin_zero(Nat.0.suc), fin_succ(Nat.0.suc, fin_zero(Nat.0))) =
        matrix2_det[F](Nat.0, a).inverse * (-matrix2_entry_01[F](Nat.0, a))
}

/// The lower-left inverse entry is the determinant inverse times the negated
/// lower-left entry.
theorem matrix2_inv_value_10[F: Field](a: Matrix[F, Nat.0.suc.suc, Nat.0.suc.suc]) {
    matrix2_inv_entry[F](Nat.0, a, fin_succ(Nat.0.suc, fin_zero(Nat.0)), fin_zero(Nat.0.suc)) =
        matrix2_det[F](Nat.0, a).inverse * (-matrix2_entry_10[F](Nat.0, a))
} by {
    matrix2_inv_entry_unfold[F](a, fin_succ(Nat.0.suc, fin_zero(Nat.0)), fin_zero(Nat.0.suc))
    matrix2_inv_entry[F](Nat.0, a, fin_succ(Nat.0.suc, fin_zero(Nat.0)), fin_zero(Nat.0.suc)) =
        matrix2_det[F](Nat.0, a).inverse * matrix2_adjugate_entry[F](Nat.0, a, fin_succ(Nat.0.suc, fin_zero(Nat.0)), fin_zero(Nat.0.suc))
    matrix2_adjugate_entry_10[F](a)
    matrix2_inv_entry[F](Nat.0, a, fin_succ(Nat.0.suc, fin_zero(Nat.0)), fin_zero(Nat.0.suc)) =
        matrix2_det[F](Nat.0, a).inverse * (-matrix2_entry_10[F](Nat.0, a))
}

/// The lower-right inverse entry is the determinant inverse times the
/// upper-left entry.
theorem matrix2_inv_value_11[F: Field](a: Matrix[F, Nat.0.suc.suc, Nat.0.suc.suc]) {
    matrix2_inv_entry[F](Nat.0, a, fin_succ(Nat.0.suc, fin_zero(Nat.0)), fin_succ(Nat.0.suc, fin_zero(Nat.0))) =
        matrix2_det[F](Nat.0, a).inverse * matrix2_entry_00[F](Nat.0, a)
} by {
    matrix2_inv_entry_unfold[F](a, fin_succ(Nat.0.suc, fin_zero(Nat.0)), fin_succ(Nat.0.suc, fin_zero(Nat.0)))
    matrix2_inv_entry[F](Nat.0, a, fin_succ(Nat.0.suc, fin_zero(Nat.0)), fin_succ(Nat.0.suc, fin_zero(Nat.0))) =
        matrix2_det[F](Nat.0, a).inverse * matrix2_adjugate_entry[F](Nat.0, a, fin_succ(Nat.0.suc, fin_zero(Nat.0)), fin_succ(Nat.0.suc, fin_zero(Nat.0)))
    matrix2_adjugate_entry_11[F](a)
    matrix2_inv_entry[F](Nat.0, a, fin_succ(Nat.0.suc, fin_zero(Nat.0)), fin_succ(Nat.0.suc, fin_zero(Nat.0))) =
        matrix2_det[F](Nat.0, a).inverse * matrix2_entry_00[F](Nat.0, a)
}

/// The inverse entry agrees with the inverse formula entry `det * a11` etc. of
/// `algebra/matrix_algebra.ac`.
theorem matrix2_inv_value_00_eq_inv2_entry_00[F: Field](a: Matrix[F, Nat.0.suc.suc, Nat.0.suc.suc]) {
    matrix2_inv_entry[F](Nat.0, a, fin_zero(Nat.0.suc), fin_zero(Nat.0.suc)) =
        inv2_entry_00[F](matrix2_entry_00[F](Nat.0, a), matrix2_entry_01[F](Nat.0, a), matrix2_entry_10[F](Nat.0, a), matrix2_entry_11[F](Nat.0, a))
} by {
    matrix2_inv_value_00[F](a)
    matrix2_det_unfold[F](a)
    det2_entry_expanded[F](matrix2_entry_00[F](Nat.0, a), matrix2_entry_01[F](Nat.0, a), matrix2_entry_10[F](Nat.0, a), matrix2_entry_11[F](Nat.0, a))
    inv2_entry_00[F](matrix2_entry_00[F](Nat.0, a), matrix2_entry_01[F](Nat.0, a), matrix2_entry_10[F](Nat.0, a), matrix2_entry_11[F](Nat.0, a)) =
        det2_entry[F](matrix2_entry_00[F](Nat.0, a), matrix2_entry_01[F](Nat.0, a), matrix2_entry_10[F](Nat.0, a), matrix2_entry_11[F](Nat.0, a)).inverse *
            matrix2_entry_11[F](Nat.0, a)
    det2_entry[F](matrix2_entry_00[F](Nat.0, a), matrix2_entry_01[F](Nat.0, a), matrix2_entry_10[F](Nat.0, a), matrix2_entry_11[F](Nat.0, a)) =
        matrix2_det[F](Nat.0, a)
    inv2_entry_00[F](matrix2_entry_00[F](Nat.0, a), matrix2_entry_01[F](Nat.0, a), matrix2_entry_10[F](Nat.0, a), matrix2_entry_11[F](Nat.0, a)) =
        matrix2_det[F](Nat.0, a).inverse * matrix2_entry_11[F](Nat.0, a)
    matrix2_inv_entry[F](Nat.0, a, fin_zero(Nat.0.suc), fin_zero(Nat.0.suc)) =
        inv2_entry_00[F](matrix2_entry_00[F](Nat.0, a), matrix2_entry_01[F](Nat.0, a), matrix2_entry_10[F](Nat.0, a), matrix2_entry_11[F](Nat.0, a))
}

/// The upper-left entry of `A * inv(A)` is one when the determinant is
/// nonzero.
theorem matrix2_inv_right_entry_00[F: Field](a: Matrix[F, Nat.0.suc.suc, Nat.0.suc.suc]) {
    matrix2_det[F](Nat.0, a) != F.0 implies
        matrix_entry[F](Nat.0.suc.suc, Nat.0.suc.suc, square_matrix_mul[F](Nat.0.suc.suc, a, matrix2_inv[F](Nat.0, a)), fin_zero(Nat.0.suc), fin_zero(Nat.0.suc)) = F.1
} by {
    if matrix2_det[F](Nat.0, a) != F.0 {
        matrix2_mul_entry_00[F](a, matrix2_inv[F](Nat.0, a))
        matrix_entry[F](Nat.0.suc.suc, Nat.0.suc.suc, square_matrix_mul[F](Nat.0.suc.suc, a, matrix2_inv[F](Nat.0, a)), fin_zero(Nat.0.suc), fin_zero(Nat.0.suc)) =
            matrix2_entry_00[F](Nat.0, a) * matrix2_entry_00[F](Nat.0, matrix2_inv[F](Nat.0, a)) +
            matrix2_entry_01[F](Nat.0, a) * matrix2_entry_10[F](Nat.0, matrix2_inv[F](Nat.0, a))
        matrix2_inv_matrix_entry[F](a, fin_zero(Nat.0.suc), fin_zero(Nat.0.suc))
        matrix2_inv_matrix_entry[F](a, fin_succ(Nat.0.suc, fin_zero(Nat.0)), fin_zero(Nat.0.suc))
        matrix2_entry_00[F](Nat.0, matrix2_inv[F](Nat.0, a)) =
            matrix2_inv_entry[F](Nat.0, a, fin_zero(Nat.0.suc), fin_zero(Nat.0.suc))
        matrix2_entry_10[F](Nat.0, matrix2_inv[F](Nat.0, a)) =
            matrix2_inv_entry[F](Nat.0, a, fin_succ(Nat.0.suc, fin_zero(Nat.0)), fin_zero(Nat.0.suc))
        matrix2_inv_value_00[F](a)
        matrix2_inv_value_10[F](a)
        matrix_entry[F](Nat.0.suc.suc, Nat.0.suc.suc, square_matrix_mul[F](Nat.0.suc.suc, a, matrix2_inv[F](Nat.0, a)), fin_zero(Nat.0.suc), fin_zero(Nat.0.suc)) =
            matrix2_entry_00[F](Nat.0, a) * (matrix2_det[F](Nat.0, a).inverse * matrix2_entry_11[F](Nat.0, a)) +
            matrix2_entry_01[F](Nat.0, a) * (matrix2_det[F](Nat.0, a).inverse * (-matrix2_entry_10[F](Nat.0, a)))
        matrix2_entry_00[F](Nat.0, a) * (matrix2_det[F](Nat.0, a).inverse * matrix2_entry_11[F](Nat.0, a)) =
            (matrix2_entry_00[F](Nat.0, a) * matrix2_entry_11[F](Nat.0, a)) * matrix2_det[F](Nat.0, a).inverse
        matrix2_entry_01[F](Nat.0, a) * (matrix2_det[F](Nat.0, a).inverse * (-matrix2_entry_10[F](Nat.0, a))) =
            (matrix2_entry_01[F](Nat.0, a) * (-matrix2_entry_10[F](Nat.0, a))) * matrix2_det[F](Nat.0, a).inverse
        matrix2_entry_01[F](Nat.0, a) * (-matrix2_entry_10[F](Nat.0, a)) =
            -(matrix2_entry_01[F](Nat.0, a) * matrix2_entry_10[F](Nat.0, a))
        matrix_entry[F](Nat.0.suc.suc, Nat.0.suc.suc, square_matrix_mul[F](Nat.0.suc.suc, a, matrix2_inv[F](Nat.0, a)), fin_zero(Nat.0.suc), fin_zero(Nat.0.suc)) =
            (matrix2_entry_00[F](Nat.0, a) * matrix2_entry_11[F](Nat.0, a)) * matrix2_det[F](Nat.0, a).inverse +
            (matrix2_entry_01[F](Nat.0, a) * (-matrix2_entry_10[F](Nat.0, a))) * matrix2_det[F](Nat.0, a).inverse
        (matrix2_entry_00[F](Nat.0, a) * matrix2_entry_11[F](Nat.0, a)) * matrix2_det[F](Nat.0, a).inverse +
            (matrix2_entry_01[F](Nat.0, a) * (-matrix2_entry_10[F](Nat.0, a))) * matrix2_det[F](Nat.0, a).inverse =
            (matrix2_entry_00[F](Nat.0, a) * matrix2_entry_11[F](Nat.0, a) +
                matrix2_entry_01[F](Nat.0, a) * (-matrix2_entry_10[F](Nat.0, a))) * matrix2_det[F](Nat.0, a).inverse
        matrix2_adjugate_mul_entry_00[F](a)
        matrix2_entry_00[F](Nat.0, a) * matrix2_entry_11[F](Nat.0, a) +
            matrix2_entry_01[F](Nat.0, a) * (-matrix2_entry_10[F](Nat.0, a)) =
            matrix2_det[F](Nat.0, a)
        matrix_entry[F](Nat.0.suc.suc, Nat.0.suc.suc, square_matrix_mul[F](Nat.0.suc.suc, a, matrix2_inv[F](Nat.0, a)), fin_zero(Nat.0.suc), fin_zero(Nat.0.suc)) =
            matrix2_det[F](Nat.0, a) * matrix2_det[F](Nat.0, a).inverse
        mul_inverse_right[F](matrix2_det[F](Nat.0, a))
        matrix2_det[F](Nat.0, a) * matrix2_det[F](Nat.0, a).inverse = F.1
        matrix_entry[F](Nat.0.suc.suc, Nat.0.suc.suc, square_matrix_mul[F](Nat.0.suc.suc, a, matrix2_inv[F](Nat.0, a)), fin_zero(Nat.0.suc), fin_zero(Nat.0.suc)) = F.1
    }
}

/// The upper-right entry of `A * inv(A)` is zero when the determinant is
/// nonzero.
theorem matrix2_inv_right_entry_01[F: Field](a: Matrix[F, Nat.0.suc.suc, Nat.0.suc.suc]) {
    matrix2_det[F](Nat.0, a) != F.0 implies
        matrix_entry[F](Nat.0.suc.suc, Nat.0.suc.suc, square_matrix_mul[F](Nat.0.suc.suc, a, matrix2_inv[F](Nat.0, a)), fin_zero(Nat.0.suc), fin_succ(Nat.0.suc, fin_zero(Nat.0))) = F.0
} by {
    if matrix2_det[F](Nat.0, a) != F.0 {
        matrix2_mul_entry_01[F](a, matrix2_inv[F](Nat.0, a))
        matrix_entry[F](Nat.0.suc.suc, Nat.0.suc.suc, square_matrix_mul[F](Nat.0.suc.suc, a, matrix2_inv[F](Nat.0, a)), fin_zero(Nat.0.suc), fin_succ(Nat.0.suc, fin_zero(Nat.0))) =
            matrix2_entry_00[F](Nat.0, a) * matrix2_entry_01[F](Nat.0, matrix2_inv[F](Nat.0, a)) +
            matrix2_entry_01[F](Nat.0, a) * matrix2_entry_11[F](Nat.0, matrix2_inv[F](Nat.0, a))
        matrix2_inv_matrix_entry[F](a, fin_zero(Nat.0.suc), fin_succ(Nat.0.suc, fin_zero(Nat.0)))
        matrix2_inv_matrix_entry[F](a, fin_succ(Nat.0.suc, fin_zero(Nat.0)), fin_succ(Nat.0.suc, fin_zero(Nat.0)))
        matrix2_entry_01[F](Nat.0, matrix2_inv[F](Nat.0, a)) =
            matrix2_inv_entry[F](Nat.0, a, fin_zero(Nat.0.suc), fin_succ(Nat.0.suc, fin_zero(Nat.0)))
        matrix2_entry_11[F](Nat.0, matrix2_inv[F](Nat.0, a)) =
            matrix2_inv_entry[F](Nat.0, a, fin_succ(Nat.0.suc, fin_zero(Nat.0)), fin_succ(Nat.0.suc, fin_zero(Nat.0)))
        matrix2_inv_value_01[F](a)
        matrix2_inv_value_11[F](a)
        matrix_entry[F](Nat.0.suc.suc, Nat.0.suc.suc, square_matrix_mul[F](Nat.0.suc.suc, a, matrix2_inv[F](Nat.0, a)), fin_zero(Nat.0.suc), fin_succ(Nat.0.suc, fin_zero(Nat.0))) =
            matrix2_entry_00[F](Nat.0, a) * (matrix2_det[F](Nat.0, a).inverse * (-matrix2_entry_01[F](Nat.0, a))) +
            matrix2_entry_01[F](Nat.0, a) * (matrix2_det[F](Nat.0, a).inverse * matrix2_entry_00[F](Nat.0, a))
        matrix2_entry_00[F](Nat.0, a) * (matrix2_det[F](Nat.0, a).inverse * (-matrix2_entry_01[F](Nat.0, a))) =
            (matrix2_entry_00[F](Nat.0, a) * (-matrix2_entry_01[F](Nat.0, a))) * matrix2_det[F](Nat.0, a).inverse
        matrix2_entry_01[F](Nat.0, a) * (matrix2_det[F](Nat.0, a).inverse * matrix2_entry_00[F](Nat.0, a)) =
            (matrix2_entry_01[F](Nat.0, a) * matrix2_entry_00[F](Nat.0, a)) * matrix2_det[F](Nat.0, a).inverse
        matrix2_entry_00[F](Nat.0, a) * (-matrix2_entry_01[F](Nat.0, a)) =
            -(matrix2_entry_00[F](Nat.0, a) * matrix2_entry_01[F](Nat.0, a))
        matrix2_entry_01[F](Nat.0, a) * matrix2_entry_00[F](Nat.0, a) =
            matrix2_entry_00[F](Nat.0, a) * matrix2_entry_01[F](Nat.0, a)
        matrix_entry[F](Nat.0.suc.suc, Nat.0.suc.suc, square_matrix_mul[F](Nat.0.suc.suc, a, matrix2_inv[F](Nat.0, a)), fin_zero(Nat.0.suc), fin_succ(Nat.0.suc, fin_zero(Nat.0))) =
            (-(matrix2_entry_00[F](Nat.0, a) * matrix2_entry_01[F](Nat.0, a))) * matrix2_det[F](Nat.0, a).inverse +
            (matrix2_entry_00[F](Nat.0, a) * matrix2_entry_01[F](Nat.0, a)) * matrix2_det[F](Nat.0, a).inverse
        (-(matrix2_entry_00[F](Nat.0, a) * matrix2_entry_01[F](Nat.0, a))) * matrix2_det[F](Nat.0, a).inverse +
            (matrix2_entry_00[F](Nat.0, a) * matrix2_entry_01[F](Nat.0, a)) * matrix2_det[F](Nat.0, a).inverse = F.0
        matrix_entry[F](Nat.0.suc.suc, Nat.0.suc.suc, square_matrix_mul[F](Nat.0.suc.suc, a, matrix2_inv[F](Nat.0, a)), fin_zero(Nat.0.suc), fin_succ(Nat.0.suc, fin_zero(Nat.0))) = F.0
    }
}

/// The lower-left entry of `A * inv(A)` is zero when the determinant is
/// nonzero.
theorem matrix2_inv_right_entry_10[F: Field](a: Matrix[F, Nat.0.suc.suc, Nat.0.suc.suc]) {
    matrix2_det[F](Nat.0, a) != F.0 implies
        matrix_entry[F](Nat.0.suc.suc, Nat.0.suc.suc, square_matrix_mul[F](Nat.0.suc.suc, a, matrix2_inv[F](Nat.0, a)), fin_succ(Nat.0.suc, fin_zero(Nat.0)), fin_zero(Nat.0.suc)) = F.0
} by {
    if matrix2_det[F](Nat.0, a) != F.0 {
        matrix2_mul_entry_10[F](a, matrix2_inv[F](Nat.0, a))
        matrix_entry[F](Nat.0.suc.suc, Nat.0.suc.suc, square_matrix_mul[F](Nat.0.suc.suc, a, matrix2_inv[F](Nat.0, a)), fin_succ(Nat.0.suc, fin_zero(Nat.0)), fin_zero(Nat.0.suc)) =
            matrix2_entry_10[F](Nat.0, a) * matrix2_entry_00[F](Nat.0, matrix2_inv[F](Nat.0, a)) +
            matrix2_entry_11[F](Nat.0, a) * matrix2_entry_10[F](Nat.0, matrix2_inv[F](Nat.0, a))
        matrix2_inv_matrix_entry[F](a, fin_zero(Nat.0.suc), fin_zero(Nat.0.suc))
        matrix2_inv_matrix_entry[F](a, fin_succ(Nat.0.suc, fin_zero(Nat.0)), fin_zero(Nat.0.suc))
        matrix2_entry_00[F](Nat.0, matrix2_inv[F](Nat.0, a)) =
            matrix2_inv_entry[F](Nat.0, a, fin_zero(Nat.0.suc), fin_zero(Nat.0.suc))
        matrix2_entry_10[F](Nat.0, matrix2_inv[F](Nat.0, a)) =
            matrix2_inv_entry[F](Nat.0, a, fin_succ(Nat.0.suc, fin_zero(Nat.0)), fin_zero(Nat.0.suc))
        matrix2_inv_value_00[F](a)
        matrix2_inv_value_10[F](a)
        matrix_entry[F](Nat.0.suc.suc, Nat.0.suc.suc, square_matrix_mul[F](Nat.0.suc.suc, a, matrix2_inv[F](Nat.0, a)), fin_succ(Nat.0.suc, fin_zero(Nat.0)), fin_zero(Nat.0.suc)) =
            matrix2_entry_10[F](Nat.0, a) * (matrix2_det[F](Nat.0, a).inverse * matrix2_entry_11[F](Nat.0, a)) +
            matrix2_entry_11[F](Nat.0, a) * (matrix2_det[F](Nat.0, a).inverse * (-matrix2_entry_10[F](Nat.0, a)))
        matrix2_entry_10[F](Nat.0, a) * (matrix2_det[F](Nat.0, a).inverse * matrix2_entry_11[F](Nat.0, a)) =
            (matrix2_entry_10[F](Nat.0, a) * matrix2_entry_11[F](Nat.0, a)) * matrix2_det[F](Nat.0, a).inverse
        matrix2_entry_11[F](Nat.0, a) * (matrix2_det[F](Nat.0, a).inverse * (-matrix2_entry_10[F](Nat.0, a))) =
            (matrix2_entry_11[F](Nat.0, a) * (-matrix2_entry_10[F](Nat.0, a))) * matrix2_det[F](Nat.0, a).inverse
        matrix2_entry_11[F](Nat.0, a) * (-matrix2_entry_10[F](Nat.0, a)) =
            -(matrix2_entry_11[F](Nat.0, a) * matrix2_entry_10[F](Nat.0, a))
        matrix2_entry_11[F](Nat.0, a) * matrix2_entry_10[F](Nat.0, a) =
            matrix2_entry_10[F](Nat.0, a) * matrix2_entry_11[F](Nat.0, a)
        matrix_entry[F](Nat.0.suc.suc, Nat.0.suc.suc, square_matrix_mul[F](Nat.0.suc.suc, a, matrix2_inv[F](Nat.0, a)), fin_succ(Nat.0.suc, fin_zero(Nat.0)), fin_zero(Nat.0.suc)) =
            (matrix2_entry_10[F](Nat.0, a) * matrix2_entry_11[F](Nat.0, a)) * matrix2_det[F](Nat.0, a).inverse +
            (-(matrix2_entry_10[F](Nat.0, a) * matrix2_entry_11[F](Nat.0, a))) * matrix2_det[F](Nat.0, a).inverse
        (matrix2_entry_10[F](Nat.0, a) * matrix2_entry_11[F](Nat.0, a)) * matrix2_det[F](Nat.0, a).inverse +
            (-(matrix2_entry_10[F](Nat.0, a) * matrix2_entry_11[F](Nat.0, a))) * matrix2_det[F](Nat.0, a).inverse = F.0
        matrix_entry[F](Nat.0.suc.suc, Nat.0.suc.suc, square_matrix_mul[F](Nat.0.suc.suc, a, matrix2_inv[F](Nat.0, a)), fin_succ(Nat.0.suc, fin_zero(Nat.0)), fin_zero(Nat.0.suc)) = F.0
    }
}

/// The lower-right entry of `A * inv(A)` is one when the determinant is
/// nonzero.
theorem matrix2_inv_right_entry_11[F: Field](a: Matrix[F, Nat.0.suc.suc, Nat.0.suc.suc]) {
    matrix2_det[F](Nat.0, a) != F.0 implies
        matrix_entry[F](Nat.0.suc.suc, Nat.0.suc.suc, square_matrix_mul[F](Nat.0.suc.suc, a, matrix2_inv[F](Nat.0, a)), fin_succ(Nat.0.suc, fin_zero(Nat.0)), fin_succ(Nat.0.suc, fin_zero(Nat.0))) = F.1
} by {
    if matrix2_det[F](Nat.0, a) != F.0 {
        matrix2_mul_entry_11[F](a, matrix2_inv[F](Nat.0, a))
        matrix_entry[F](Nat.0.suc.suc, Nat.0.suc.suc, square_matrix_mul[F](Nat.0.suc.suc, a, matrix2_inv[F](Nat.0, a)), fin_succ(Nat.0.suc, fin_zero(Nat.0)), fin_succ(Nat.0.suc, fin_zero(Nat.0))) =
            matrix2_entry_10[F](Nat.0, a) * matrix2_entry_01[F](Nat.0, matrix2_inv[F](Nat.0, a)) +
            matrix2_entry_11[F](Nat.0, a) * matrix2_entry_11[F](Nat.0, matrix2_inv[F](Nat.0, a))
        matrix2_inv_matrix_entry[F](a, fin_zero(Nat.0.suc), fin_succ(Nat.0.suc, fin_zero(Nat.0)))
        matrix2_inv_matrix_entry[F](a, fin_succ(Nat.0.suc, fin_zero(Nat.0)), fin_succ(Nat.0.suc, fin_zero(Nat.0)))
        matrix2_entry_01[F](Nat.0, matrix2_inv[F](Nat.0, a)) =
            matrix2_inv_entry[F](Nat.0, a, fin_zero(Nat.0.suc), fin_succ(Nat.0.suc, fin_zero(Nat.0)))
        matrix2_entry_11[F](Nat.0, matrix2_inv[F](Nat.0, a)) =
            matrix2_inv_entry[F](Nat.0, a, fin_succ(Nat.0.suc, fin_zero(Nat.0)), fin_succ(Nat.0.suc, fin_zero(Nat.0)))
        matrix2_inv_value_01[F](a)
        matrix2_inv_value_11[F](a)
        matrix_entry[F](Nat.0.suc.suc, Nat.0.suc.suc, square_matrix_mul[F](Nat.0.suc.suc, a, matrix2_inv[F](Nat.0, a)), fin_succ(Nat.0.suc, fin_zero(Nat.0)), fin_succ(Nat.0.suc, fin_zero(Nat.0))) =
            matrix2_entry_10[F](Nat.0, a) * (matrix2_det[F](Nat.0, a).inverse * (-matrix2_entry_01[F](Nat.0, a))) +
            matrix2_entry_11[F](Nat.0, a) * (matrix2_det[F](Nat.0, a).inverse * matrix2_entry_00[F](Nat.0, a))
        matrix2_entry_10[F](Nat.0, a) * (matrix2_det[F](Nat.0, a).inverse * (-matrix2_entry_01[F](Nat.0, a))) =
            (matrix2_entry_10[F](Nat.0, a) * (-matrix2_entry_01[F](Nat.0, a))) * matrix2_det[F](Nat.0, a).inverse
        matrix2_entry_11[F](Nat.0, a) * (matrix2_det[F](Nat.0, a).inverse * matrix2_entry_00[F](Nat.0, a)) =
            (matrix2_entry_11[F](Nat.0, a) * matrix2_entry_00[F](Nat.0, a)) * matrix2_det[F](Nat.0, a).inverse
        matrix2_entry_10[F](Nat.0, a) * (-matrix2_entry_01[F](Nat.0, a)) =
            -(matrix2_entry_10[F](Nat.0, a) * matrix2_entry_01[F](Nat.0, a))
        matrix2_entry_10[F](Nat.0, a) * matrix2_entry_01[F](Nat.0, a) =
            matrix2_entry_01[F](Nat.0, a) * matrix2_entry_10[F](Nat.0, a)
        matrix2_entry_11[F](Nat.0, a) * matrix2_entry_00[F](Nat.0, a) =
            matrix2_entry_00[F](Nat.0, a) * matrix2_entry_11[F](Nat.0, a)
        matrix_entry[F](Nat.0.suc.suc, Nat.0.suc.suc, square_matrix_mul[F](Nat.0.suc.suc, a, matrix2_inv[F](Nat.0, a)), fin_succ(Nat.0.suc, fin_zero(Nat.0)), fin_succ(Nat.0.suc, fin_zero(Nat.0))) =
            (-(matrix2_entry_01[F](Nat.0, a) * matrix2_entry_10[F](Nat.0, a))) * matrix2_det[F](Nat.0, a).inverse +
            (matrix2_entry_00[F](Nat.0, a) * matrix2_entry_11[F](Nat.0, a)) * matrix2_det[F](Nat.0, a).inverse
        (-(matrix2_entry_01[F](Nat.0, a) * matrix2_entry_10[F](Nat.0, a))) * matrix2_det[F](Nat.0, a).inverse =
            -((matrix2_entry_01[F](Nat.0, a) * matrix2_entry_10[F](Nat.0, a)) * matrix2_det[F](Nat.0, a).inverse)
        -((matrix2_entry_01[F](Nat.0, a) * matrix2_entry_10[F](Nat.0, a)) * matrix2_det[F](Nat.0, a).inverse) +
            (matrix2_entry_00[F](Nat.0, a) * matrix2_entry_11[F](Nat.0, a)) * matrix2_det[F](Nat.0, a).inverse =
            (matrix2_entry_00[F](Nat.0, a) * matrix2_entry_11[F](Nat.0, a) +
                -(matrix2_entry_01[F](Nat.0, a) * matrix2_entry_10[F](Nat.0, a))) * matrix2_det[F](Nat.0, a).inverse
        (matrix2_entry_00[F](Nat.0, a) * matrix2_entry_11[F](Nat.0, a) +
                -(matrix2_entry_01[F](Nat.0, a) * matrix2_entry_10[F](Nat.0, a))) * matrix2_det[F](Nat.0, a).inverse =
            (matrix2_entry_00[F](Nat.0, a) * matrix2_entry_11[F](Nat.0, a) -
                matrix2_entry_01[F](Nat.0, a) * matrix2_entry_10[F](Nat.0, a)) * matrix2_det[F](Nat.0, a).inverse
        matrix2_adjugate_mul_entry_11[F](a)
        matrix2_entry_00[F](Nat.0, a) * matrix2_entry_11[F](Nat.0, a) -
            matrix2_entry_01[F](Nat.0, a) * matrix2_entry_10[F](Nat.0, a) =
            matrix2_det[F](Nat.0, a)
        matrix_entry[F](Nat.0.suc.suc, Nat.0.suc.suc, square_matrix_mul[F](Nat.0.suc.suc, a, matrix2_inv[F](Nat.0, a)), fin_succ(Nat.0.suc, fin_zero(Nat.0)), fin_succ(Nat.0.suc, fin_zero(Nat.0))) =
            matrix2_det[F](Nat.0, a) * matrix2_det[F](Nat.0, a).inverse
        mul_inverse_right[F](matrix2_det[F](Nat.0, a))
        matrix2_det[F](Nat.0, a) * matrix2_det[F](Nat.0, a).inverse = F.1
        matrix_entry[F](Nat.0.suc.suc, Nat.0.suc.suc, square_matrix_mul[F](Nat.0.suc.suc, a, matrix2_inv[F](Nat.0, a)), fin_succ(Nat.0.suc, fin_zero(Nat.0)), fin_succ(Nat.0.suc, fin_zero(Nat.0))) = F.1
    }
}

/// Multiplying by the inverse on the right gives the identity, when the
/// determinant is nonzero.
theorem matrix2_inv_right[F: Field](a: Matrix[F, Nat.0.suc.suc, Nat.0.suc.suc]) {
    matrix2_det[F](Nat.0, a) != F.0 implies
        square_matrix_mul[F](Nat.0.suc.suc, a, matrix2_inv[F](Nat.0, a)) = matrix_one[F](Nat.0.suc.suc)
} by {
    if matrix2_det[F](Nat.0, a) != F.0 {
        let lhs = square_matrix_mul[F](Nat.0.suc.suc, a, matrix2_inv[F](Nat.0, a))
        forall(i: Fin[Nat.0.suc.suc], j: Fin[Nat.0.suc.suc]) {
            if i.value = Nat.0 {
                if j.value = Nat.0 {
                    fin_two_zero_value_two
                    i.value = fin_zero(Nat.0.suc).value
                    ext(Nat.0.suc.suc, i, fin_zero(Nat.0.suc))
                    i = fin_zero(Nat.0.suc)
                    j.value = fin_zero(Nat.0.suc).value
                    ext(Nat.0.suc.suc, j, fin_zero(Nat.0.suc))
                    j = fin_zero(Nat.0.suc)
                    matrix2_det[F](Nat.0, a) != F.0
                    matrix2_inv_right_entry_00[F](a)
                    matrix_entry[F](Nat.0.suc.suc, Nat.0.suc.suc, square_matrix_mul[F](Nat.0.suc.suc, a, matrix2_inv[F](Nat.0, a)), fin_zero(Nat.0.suc), fin_zero(Nat.0.suc)) = F.1
                    matrix_entry[F](Nat.0.suc.suc, Nat.0.suc.suc, lhs, fin_zero(Nat.0.suc), fin_zero(Nat.0.suc)) = F.1
                    matrix_entry[F](Nat.0.suc.suc, Nat.0.suc.suc, lhs, i, j) = F.1
                    j = i
                    matrix_entry_one_diag[F](Nat.0.suc.suc, i)
                    matrix_entry[F](Nat.0.suc.suc, Nat.0.suc.suc, matrix_one[F](Nat.0.suc.suc), i, j) = F.1
                    matrix_entry[F](Nat.0.suc.suc, Nat.0.suc.suc, lhs, i, j) =
                        matrix_entry[F](Nat.0.suc.suc, Nat.0.suc.suc, matrix_one[F](Nat.0.suc.suc), i, j)
                } else {
                    fin_two_zero_value_two
                    i.value = fin_zero(Nat.0.suc).value
                    ext(Nat.0.suc.suc, i, fin_zero(Nat.0.suc))
                    i = fin_zero(Nat.0.suc)
                    fin_two_one_value_two
                    j.value = fin_succ(Nat.0.suc, fin_zero(Nat.0)).value
                    ext(Nat.0.suc.suc, j, fin_succ(Nat.0.suc, fin_zero(Nat.0)))
                    j = fin_succ(Nat.0.suc, fin_zero(Nat.0))
                    matrix2_det[F](Nat.0, a) != F.0
                    matrix2_inv_right_entry_01[F](a)
                    matrix_entry[F](Nat.0.suc.suc, Nat.0.suc.suc, square_matrix_mul[F](Nat.0.suc.suc, a, matrix2_inv[F](Nat.0, a)), fin_zero(Nat.0.suc), fin_succ(Nat.0.suc, fin_zero(Nat.0))) = F.0
                    matrix_entry[F](Nat.0.suc.suc, Nat.0.suc.suc, lhs, fin_zero(Nat.0.suc), fin_succ(Nat.0.suc, fin_zero(Nat.0))) = F.0
                    matrix_entry[F](Nat.0.suc.suc, Nat.0.suc.suc, lhs, i, j) = F.0
                    fin_two_zero_ne_one
                    i != j
                    matrix_entry_one_off_diag[F](Nat.0.suc.suc, i, j)
                    matrix_entry[F](Nat.0.suc.suc, Nat.0.suc.suc, matrix_one[F](Nat.0.suc.suc), i, j) = F.0
                    matrix_entry[F](Nat.0.suc.suc, Nat.0.suc.suc, lhs, i, j) =
                        matrix_entry[F](Nat.0.suc.suc, Nat.0.suc.suc, matrix_one[F](Nat.0.suc.suc), i, j)
                }
            } else {
                if j.value = Nat.0 {
                    fin_two_one_value_two
                    i.value = fin_succ(Nat.0.suc, fin_zero(Nat.0)).value
                    ext(Nat.0.suc.suc, i, fin_succ(Nat.0.suc, fin_zero(Nat.0)))
                    i = fin_succ(Nat.0.suc, fin_zero(Nat.0))
                    fin_two_zero_value_two
                    j.value = fin_zero(Nat.0.suc).value
                    ext(Nat.0.suc.suc, j, fin_zero(Nat.0.suc))
                    j = fin_zero(Nat.0.suc)
                    matrix2_det[F](Nat.0, a) != F.0
                    matrix2_inv_right_entry_10[F](a)
                    matrix_entry[F](Nat.0.suc.suc, Nat.0.suc.suc, square_matrix_mul[F](Nat.0.suc.suc, a, matrix2_inv[F](Nat.0, a)), fin_succ(Nat.0.suc, fin_zero(Nat.0)), fin_zero(Nat.0.suc)) = F.0
                    matrix_entry[F](Nat.0.suc.suc, Nat.0.suc.suc, lhs, fin_succ(Nat.0.suc, fin_zero(Nat.0)), fin_zero(Nat.0.suc)) = F.0
                    matrix_entry[F](Nat.0.suc.suc, Nat.0.suc.suc, lhs, i, j) = F.0
                    fin_two_one_ne_zero
                    i != j
                    matrix_entry_one_off_diag[F](Nat.0.suc.suc, i, j)
                    matrix_entry[F](Nat.0.suc.suc, Nat.0.suc.suc, matrix_one[F](Nat.0.suc.suc), i, j) = F.0
                    matrix_entry[F](Nat.0.suc.suc, Nat.0.suc.suc, lhs, i, j) =
                        matrix_entry[F](Nat.0.suc.suc, Nat.0.suc.suc, matrix_one[F](Nat.0.suc.suc), i, j)
                } else {
                    fin_two_one_value_two
                    i.value = fin_succ(Nat.0.suc, fin_zero(Nat.0)).value
                    ext(Nat.0.suc.suc, i, fin_succ(Nat.0.suc, fin_zero(Nat.0)))
                    i = fin_succ(Nat.0.suc, fin_zero(Nat.0))
                    j.value = fin_succ(Nat.0.suc, fin_zero(Nat.0)).value
                    ext(Nat.0.suc.suc, j, fin_succ(Nat.0.suc, fin_zero(Nat.0)))
                    j = fin_succ(Nat.0.suc, fin_zero(Nat.0))
                    matrix2_det[F](Nat.0, a) != F.0
                    matrix2_inv_right_entry_11[F](a)
                    matrix_entry[F](Nat.0.suc.suc, Nat.0.suc.suc, square_matrix_mul[F](Nat.0.suc.suc, a, matrix2_inv[F](Nat.0, a)), fin_succ(Nat.0.suc, fin_zero(Nat.0)), fin_succ(Nat.0.suc, fin_zero(Nat.0))) = F.1
                    matrix_entry[F](Nat.0.suc.suc, Nat.0.suc.suc, lhs, fin_succ(Nat.0.suc, fin_zero(Nat.0)), fin_succ(Nat.0.suc, fin_zero(Nat.0))) = F.1
                    matrix_entry[F](Nat.0.suc.suc, Nat.0.suc.suc, lhs, i, j) = F.1
                    j = i
                    matrix_entry_one_diag[F](Nat.0.suc.suc, i)
                    matrix_entry[F](Nat.0.suc.suc, Nat.0.suc.suc, matrix_one[F](Nat.0.suc.suc), i, j) = F.1
                    matrix_entry[F](Nat.0.suc.suc, Nat.0.suc.suc, lhs, i, j) =
                        matrix_entry[F](Nat.0.suc.suc, Nat.0.suc.suc, matrix_one[F](Nat.0.suc.suc), i, j)
                }
            }
            matrix_entry[F](Nat.0.suc.suc, Nat.0.suc.suc, lhs, i, j) =
                matrix_entry[F](Nat.0.suc.suc, Nat.0.suc.suc, matrix_one[F](Nat.0.suc.suc), i, j)
        }
        matrix_ext[F](Nat.0.suc.suc, Nat.0.suc.suc, lhs, matrix_one[F](Nat.0.suc.suc))
        lhs = matrix_one[F](Nat.0.suc.suc)
        square_matrix_mul[F](Nat.0.suc.suc, a, matrix2_inv[F](Nat.0, a)) = matrix_one[F](Nat.0.suc.suc)
    }
}

// ---------------------------------------------------------------------------
// Multiplying by the inverse on the left
// ---------------------------------------------------------------------------

/// The upper-left entry of `inv(A) * A` is one when the determinant is
/// nonzero.
theorem matrix2_inv_left_entry_00[F: Field](a: Matrix[F, Nat.0.suc.suc, Nat.0.suc.suc]) {
    matrix2_det[F](Nat.0, a) != F.0 implies
        matrix_entry[F](Nat.0.suc.suc, Nat.0.suc.suc, square_matrix_mul[F](Nat.0.suc.suc, matrix2_inv[F](Nat.0, a), a), fin_zero(Nat.0.suc), fin_zero(Nat.0.suc)) = F.1
} by {
    if matrix2_det[F](Nat.0, a) != F.0 {
        matrix2_mul_entry_00[F](matrix2_inv[F](Nat.0, a), a)
        matrix_entry[F](Nat.0.suc.suc, Nat.0.suc.suc, square_matrix_mul[F](Nat.0.suc.suc, matrix2_inv[F](Nat.0, a), a), fin_zero(Nat.0.suc), fin_zero(Nat.0.suc)) =
            matrix2_entry_00[F](Nat.0, matrix2_inv[F](Nat.0, a)) * matrix2_entry_00[F](Nat.0, a) +
            matrix2_entry_01[F](Nat.0, matrix2_inv[F](Nat.0, a)) * matrix2_entry_10[F](Nat.0, a)
        matrix2_inv_matrix_entry[F](a, fin_zero(Nat.0.suc), fin_zero(Nat.0.suc))
        matrix2_inv_matrix_entry[F](a, fin_zero(Nat.0.suc), fin_succ(Nat.0.suc, fin_zero(Nat.0)))
        matrix2_entry_00[F](Nat.0, matrix2_inv[F](Nat.0, a)) =
            matrix2_inv_entry[F](Nat.0, a, fin_zero(Nat.0.suc), fin_zero(Nat.0.suc))
        matrix2_entry_01[F](Nat.0, matrix2_inv[F](Nat.0, a)) =
            matrix2_inv_entry[F](Nat.0, a, fin_zero(Nat.0.suc), fin_succ(Nat.0.suc, fin_zero(Nat.0)))
        matrix2_inv_value_00[F](a)
        matrix2_inv_value_01[F](a)
        matrix_entry[F](Nat.0.suc.suc, Nat.0.suc.suc, square_matrix_mul[F](Nat.0.suc.suc, matrix2_inv[F](Nat.0, a), a), fin_zero(Nat.0.suc), fin_zero(Nat.0.suc)) =
            (matrix2_det[F](Nat.0, a).inverse * matrix2_entry_11[F](Nat.0, a)) * matrix2_entry_00[F](Nat.0, a) +
            (matrix2_det[F](Nat.0, a).inverse * (-matrix2_entry_01[F](Nat.0, a))) * matrix2_entry_10[F](Nat.0, a)
        (matrix2_det[F](Nat.0, a).inverse * matrix2_entry_11[F](Nat.0, a)) * matrix2_entry_00[F](Nat.0, a) =
            matrix2_det[F](Nat.0, a).inverse * (matrix2_entry_11[F](Nat.0, a) * matrix2_entry_00[F](Nat.0, a))
        (matrix2_det[F](Nat.0, a).inverse * (-matrix2_entry_01[F](Nat.0, a))) * matrix2_entry_10[F](Nat.0, a) =
            matrix2_det[F](Nat.0, a).inverse * ((-matrix2_entry_01[F](Nat.0, a)) * matrix2_entry_10[F](Nat.0, a))
        (-matrix2_entry_01[F](Nat.0, a)) * matrix2_entry_10[F](Nat.0, a) =
            -(matrix2_entry_01[F](Nat.0, a) * matrix2_entry_10[F](Nat.0, a))
        matrix2_entry_11[F](Nat.0, a) * matrix2_entry_00[F](Nat.0, a) =
            matrix2_entry_00[F](Nat.0, a) * matrix2_entry_11[F](Nat.0, a)
        matrix_entry[F](Nat.0.suc.suc, Nat.0.suc.suc, square_matrix_mul[F](Nat.0.suc.suc, matrix2_inv[F](Nat.0, a), a), fin_zero(Nat.0.suc), fin_zero(Nat.0.suc)) =
            matrix2_det[F](Nat.0, a).inverse *
                (matrix2_entry_11[F](Nat.0, a) * matrix2_entry_00[F](Nat.0, a) +
                    -(matrix2_entry_01[F](Nat.0, a) * matrix2_entry_10[F](Nat.0, a)))
        matrix2_det[F](Nat.0, a).inverse *
            (matrix2_entry_11[F](Nat.0, a) * matrix2_entry_00[F](Nat.0, a) +
                -(matrix2_entry_01[F](Nat.0, a) * matrix2_entry_10[F](Nat.0, a))) =
            matrix2_det[F](Nat.0, a).inverse *
                (matrix2_entry_00[F](Nat.0, a) * matrix2_entry_11[F](Nat.0, a) -
                    matrix2_entry_01[F](Nat.0, a) * matrix2_entry_10[F](Nat.0, a))
        matrix_entry[F](Nat.0.suc.suc, Nat.0.suc.suc, square_matrix_mul[F](Nat.0.suc.suc, matrix2_inv[F](Nat.0, a), a), fin_zero(Nat.0.suc), fin_zero(Nat.0.suc)) =
            matrix2_det[F](Nat.0, a).inverse *
                (matrix2_entry_00[F](Nat.0, a) * matrix2_entry_11[F](Nat.0, a) -
                    matrix2_entry_01[F](Nat.0, a) * matrix2_entry_10[F](Nat.0, a))
        matrix2_det_unfold[F](a)
        matrix_entry[F](Nat.0.suc.suc, Nat.0.suc.suc, square_matrix_mul[F](Nat.0.suc.suc, matrix2_inv[F](Nat.0, a), a), fin_zero(Nat.0.suc), fin_zero(Nat.0.suc)) =
            matrix2_det[F](Nat.0, a).inverse * matrix2_det[F](Nat.0, a)
        mul_inverse_left[F](matrix2_det[F](Nat.0, a))
        matrix2_det[F](Nat.0, a).inverse * matrix2_det[F](Nat.0, a) = F.1
        matrix_entry[F](Nat.0.suc.suc, Nat.0.suc.suc, square_matrix_mul[F](Nat.0.suc.suc, matrix2_inv[F](Nat.0, a), a), fin_zero(Nat.0.suc), fin_zero(Nat.0.suc)) = F.1
    }
}

/// The upper-right entry of `inv(A) * A` is zero when the determinant is
/// nonzero.
theorem matrix2_inv_left_entry_01[F: Field](a: Matrix[F, Nat.0.suc.suc, Nat.0.suc.suc]) {
    matrix2_det[F](Nat.0, a) != F.0 implies
        matrix_entry[F](Nat.0.suc.suc, Nat.0.suc.suc, square_matrix_mul[F](Nat.0.suc.suc, matrix2_inv[F](Nat.0, a), a), fin_zero(Nat.0.suc), fin_succ(Nat.0.suc, fin_zero(Nat.0))) = F.0
} by {
    if matrix2_det[F](Nat.0, a) != F.0 {
        matrix2_mul_entry_01[F](matrix2_inv[F](Nat.0, a), a)
        matrix_entry[F](Nat.0.suc.suc, Nat.0.suc.suc, square_matrix_mul[F](Nat.0.suc.suc, matrix2_inv[F](Nat.0, a), a), fin_zero(Nat.0.suc), fin_succ(Nat.0.suc, fin_zero(Nat.0))) =
            matrix2_entry_00[F](Nat.0, matrix2_inv[F](Nat.0, a)) * matrix2_entry_01[F](Nat.0, a) +
            matrix2_entry_01[F](Nat.0, matrix2_inv[F](Nat.0, a)) * matrix2_entry_11[F](Nat.0, a)
        matrix2_inv_matrix_entry[F](a, fin_zero(Nat.0.suc), fin_zero(Nat.0.suc))
        matrix2_inv_matrix_entry[F](a, fin_zero(Nat.0.suc), fin_succ(Nat.0.suc, fin_zero(Nat.0)))
        matrix2_entry_00[F](Nat.0, matrix2_inv[F](Nat.0, a)) =
            matrix2_inv_entry[F](Nat.0, a, fin_zero(Nat.0.suc), fin_zero(Nat.0.suc))
        matrix2_entry_01[F](Nat.0, matrix2_inv[F](Nat.0, a)) =
            matrix2_inv_entry[F](Nat.0, a, fin_zero(Nat.0.suc), fin_succ(Nat.0.suc, fin_zero(Nat.0)))
        matrix2_inv_value_00[F](a)
        matrix2_inv_value_01[F](a)
        matrix_entry[F](Nat.0.suc.suc, Nat.0.suc.suc, square_matrix_mul[F](Nat.0.suc.suc, matrix2_inv[F](Nat.0, a), a), fin_zero(Nat.0.suc), fin_succ(Nat.0.suc, fin_zero(Nat.0))) =
            (matrix2_det[F](Nat.0, a).inverse * matrix2_entry_11[F](Nat.0, a)) * matrix2_entry_01[F](Nat.0, a) +
            (matrix2_det[F](Nat.0, a).inverse * (-matrix2_entry_01[F](Nat.0, a))) * matrix2_entry_11[F](Nat.0, a)
        (matrix2_det[F](Nat.0, a).inverse * matrix2_entry_11[F](Nat.0, a)) * matrix2_entry_01[F](Nat.0, a) =
            matrix2_det[F](Nat.0, a).inverse * (matrix2_entry_11[F](Nat.0, a) * matrix2_entry_01[F](Nat.0, a))
        (matrix2_det[F](Nat.0, a).inverse * (-matrix2_entry_01[F](Nat.0, a))) * matrix2_entry_11[F](Nat.0, a) =
            matrix2_det[F](Nat.0, a).inverse * ((-matrix2_entry_01[F](Nat.0, a)) * matrix2_entry_11[F](Nat.0, a))
        (-matrix2_entry_01[F](Nat.0, a)) * matrix2_entry_11[F](Nat.0, a) =
            -(matrix2_entry_01[F](Nat.0, a) * matrix2_entry_11[F](Nat.0, a))
        matrix2_entry_11[F](Nat.0, a) * matrix2_entry_01[F](Nat.0, a) =
            matrix2_entry_01[F](Nat.0, a) * matrix2_entry_11[F](Nat.0, a)
        matrix_entry[F](Nat.0.suc.suc, Nat.0.suc.suc, square_matrix_mul[F](Nat.0.suc.suc, matrix2_inv[F](Nat.0, a), a), fin_zero(Nat.0.suc), fin_succ(Nat.0.suc, fin_zero(Nat.0))) =
            matrix2_det[F](Nat.0, a).inverse *
                (matrix2_entry_11[F](Nat.0, a) * matrix2_entry_01[F](Nat.0, a) +
                    -(matrix2_entry_01[F](Nat.0, a) * matrix2_entry_11[F](Nat.0, a)))
        matrix2_det[F](Nat.0, a).inverse *
            (matrix2_entry_11[F](Nat.0, a) * matrix2_entry_01[F](Nat.0, a) +
                -(matrix2_entry_01[F](Nat.0, a) * matrix2_entry_11[F](Nat.0, a))) =
            matrix2_det[F](Nat.0, a).inverse *
                (matrix2_entry_01[F](Nat.0, a) * matrix2_entry_11[F](Nat.0, a) -
                    matrix2_entry_01[F](Nat.0, a) * matrix2_entry_11[F](Nat.0, a))
        matrix_entry[F](Nat.0.suc.suc, Nat.0.suc.suc, square_matrix_mul[F](Nat.0.suc.suc, matrix2_inv[F](Nat.0, a), a), fin_zero(Nat.0.suc), fin_succ(Nat.0.suc, fin_zero(Nat.0))) =
            matrix2_det[F](Nat.0, a).inverse *
                (matrix2_entry_01[F](Nat.0, a) * matrix2_entry_11[F](Nat.0, a) -
                    matrix2_entry_01[F](Nat.0, a) * matrix2_entry_11[F](Nat.0, a))
        matrix2_entry_01[F](Nat.0, a) * matrix2_entry_11[F](Nat.0, a) -
            matrix2_entry_01[F](Nat.0, a) * matrix2_entry_11[F](Nat.0, a) = F.0
        matrix_entry[F](Nat.0.suc.suc, Nat.0.suc.suc, square_matrix_mul[F](Nat.0.suc.suc, matrix2_inv[F](Nat.0, a), a), fin_zero(Nat.0.suc), fin_succ(Nat.0.suc, fin_zero(Nat.0))) =
            matrix2_det[F](Nat.0, a).inverse * F.0
        matrix2_det[F](Nat.0, a).inverse * F.0 = F.0
        matrix_entry[F](Nat.0.suc.suc, Nat.0.suc.suc, square_matrix_mul[F](Nat.0.suc.suc, matrix2_inv[F](Nat.0, a), a), fin_zero(Nat.0.suc), fin_succ(Nat.0.suc, fin_zero(Nat.0))) = F.0
    }
}

/// The lower-left entry of `inv(A) * A` is zero when the determinant is
/// nonzero.
theorem matrix2_inv_left_entry_10[F: Field](a: Matrix[F, Nat.0.suc.suc, Nat.0.suc.suc]) {
    matrix2_det[F](Nat.0, a) != F.0 implies
        matrix_entry[F](Nat.0.suc.suc, Nat.0.suc.suc, square_matrix_mul[F](Nat.0.suc.suc, matrix2_inv[F](Nat.0, a), a), fin_succ(Nat.0.suc, fin_zero(Nat.0)), fin_zero(Nat.0.suc)) = F.0
} by {
    if matrix2_det[F](Nat.0, a) != F.0 {
        matrix2_mul_entry_10[F](matrix2_inv[F](Nat.0, a), a)
        matrix_entry[F](Nat.0.suc.suc, Nat.0.suc.suc, square_matrix_mul[F](Nat.0.suc.suc, matrix2_inv[F](Nat.0, a), a), fin_succ(Nat.0.suc, fin_zero(Nat.0)), fin_zero(Nat.0.suc)) =
            matrix2_entry_10[F](Nat.0, matrix2_inv[F](Nat.0, a)) * matrix2_entry_00[F](Nat.0, a) +
            matrix2_entry_11[F](Nat.0, matrix2_inv[F](Nat.0, a)) * matrix2_entry_10[F](Nat.0, a)
        matrix2_inv_matrix_entry[F](a, fin_succ(Nat.0.suc, fin_zero(Nat.0)), fin_zero(Nat.0.suc))
        matrix2_inv_matrix_entry[F](a, fin_succ(Nat.0.suc, fin_zero(Nat.0)), fin_succ(Nat.0.suc, fin_zero(Nat.0)))
        matrix2_entry_10[F](Nat.0, matrix2_inv[F](Nat.0, a)) =
            matrix2_inv_entry[F](Nat.0, a, fin_succ(Nat.0.suc, fin_zero(Nat.0)), fin_zero(Nat.0.suc))
        matrix2_entry_11[F](Nat.0, matrix2_inv[F](Nat.0, a)) =
            matrix2_inv_entry[F](Nat.0, a, fin_succ(Nat.0.suc, fin_zero(Nat.0)), fin_succ(Nat.0.suc, fin_zero(Nat.0)))
        matrix2_inv_value_10[F](a)
        matrix2_inv_value_11[F](a)
        matrix_entry[F](Nat.0.suc.suc, Nat.0.suc.suc, square_matrix_mul[F](Nat.0.suc.suc, matrix2_inv[F](Nat.0, a), a), fin_succ(Nat.0.suc, fin_zero(Nat.0)), fin_zero(Nat.0.suc)) =
            (matrix2_det[F](Nat.0, a).inverse * (-matrix2_entry_10[F](Nat.0, a))) * matrix2_entry_00[F](Nat.0, a) +
            (matrix2_det[F](Nat.0, a).inverse * matrix2_entry_00[F](Nat.0, a)) * matrix2_entry_10[F](Nat.0, a)
        (matrix2_det[F](Nat.0, a).inverse * (-matrix2_entry_10[F](Nat.0, a))) * matrix2_entry_00[F](Nat.0, a) =
            matrix2_det[F](Nat.0, a).inverse * ((-matrix2_entry_10[F](Nat.0, a)) * matrix2_entry_00[F](Nat.0, a))
        (matrix2_det[F](Nat.0, a).inverse * matrix2_entry_00[F](Nat.0, a)) * matrix2_entry_10[F](Nat.0, a) =
            matrix2_det[F](Nat.0, a).inverse * (matrix2_entry_00[F](Nat.0, a) * matrix2_entry_10[F](Nat.0, a))
        (-matrix2_entry_10[F](Nat.0, a)) * matrix2_entry_00[F](Nat.0, a) =
            -(matrix2_entry_10[F](Nat.0, a) * matrix2_entry_00[F](Nat.0, a))
        matrix2_entry_10[F](Nat.0, a) * matrix2_entry_00[F](Nat.0, a) =
            matrix2_entry_00[F](Nat.0, a) * matrix2_entry_10[F](Nat.0, a)
        matrix_entry[F](Nat.0.suc.suc, Nat.0.suc.suc, square_matrix_mul[F](Nat.0.suc.suc, matrix2_inv[F](Nat.0, a), a), fin_succ(Nat.0.suc, fin_zero(Nat.0)), fin_zero(Nat.0.suc)) =
            matrix2_det[F](Nat.0, a).inverse *
                (-(matrix2_entry_00[F](Nat.0, a) * matrix2_entry_10[F](Nat.0, a)) +
                    matrix2_entry_00[F](Nat.0, a) * matrix2_entry_10[F](Nat.0, a))
        -(matrix2_entry_00[F](Nat.0, a) * matrix2_entry_10[F](Nat.0, a)) +
            matrix2_entry_00[F](Nat.0, a) * matrix2_entry_10[F](Nat.0, a) = F.0
        matrix_entry[F](Nat.0.suc.suc, Nat.0.suc.suc, square_matrix_mul[F](Nat.0.suc.suc, matrix2_inv[F](Nat.0, a), a), fin_succ(Nat.0.suc, fin_zero(Nat.0)), fin_zero(Nat.0.suc)) =
            matrix2_det[F](Nat.0, a).inverse * F.0
        matrix2_det[F](Nat.0, a).inverse * F.0 = F.0
        matrix_entry[F](Nat.0.suc.suc, Nat.0.suc.suc, square_matrix_mul[F](Nat.0.suc.suc, matrix2_inv[F](Nat.0, a), a), fin_succ(Nat.0.suc, fin_zero(Nat.0)), fin_zero(Nat.0.suc)) = F.0
    }
}

/// The lower-right entry of `inv(A) * A` is one when the determinant is
/// nonzero.
theorem matrix2_inv_left_entry_11[F: Field](a: Matrix[F, Nat.0.suc.suc, Nat.0.suc.suc]) {
    matrix2_det[F](Nat.0, a) != F.0 implies
        matrix_entry[F](Nat.0.suc.suc, Nat.0.suc.suc, square_matrix_mul[F](Nat.0.suc.suc, matrix2_inv[F](Nat.0, a), a), fin_succ(Nat.0.suc, fin_zero(Nat.0)), fin_succ(Nat.0.suc, fin_zero(Nat.0))) = F.1
} by {
    if matrix2_det[F](Nat.0, a) != F.0 {
        matrix2_mul_entry_11[F](matrix2_inv[F](Nat.0, a), a)
        matrix_entry[F](Nat.0.suc.suc, Nat.0.suc.suc, square_matrix_mul[F](Nat.0.suc.suc, matrix2_inv[F](Nat.0, a), a), fin_succ(Nat.0.suc, fin_zero(Nat.0)), fin_succ(Nat.0.suc, fin_zero(Nat.0))) =
            matrix2_entry_10[F](Nat.0, matrix2_inv[F](Nat.0, a)) * matrix2_entry_01[F](Nat.0, a) +
            matrix2_entry_11[F](Nat.0, matrix2_inv[F](Nat.0, a)) * matrix2_entry_11[F](Nat.0, a)
        matrix2_inv_matrix_entry[F](a, fin_succ(Nat.0.suc, fin_zero(Nat.0)), fin_zero(Nat.0.suc))
        matrix2_inv_matrix_entry[F](a, fin_succ(Nat.0.suc, fin_zero(Nat.0)), fin_succ(Nat.0.suc, fin_zero(Nat.0)))
        matrix2_entry_10[F](Nat.0, matrix2_inv[F](Nat.0, a)) =
            matrix2_inv_entry[F](Nat.0, a, fin_succ(Nat.0.suc, fin_zero(Nat.0)), fin_zero(Nat.0.suc))
        matrix2_entry_11[F](Nat.0, matrix2_inv[F](Nat.0, a)) =
            matrix2_inv_entry[F](Nat.0, a, fin_succ(Nat.0.suc, fin_zero(Nat.0)), fin_succ(Nat.0.suc, fin_zero(Nat.0)))
        matrix2_inv_value_10[F](a)
        matrix2_inv_value_11[F](a)
        matrix_entry[F](Nat.0.suc.suc, Nat.0.suc.suc, square_matrix_mul[F](Nat.0.suc.suc, matrix2_inv[F](Nat.0, a), a), fin_succ(Nat.0.suc, fin_zero(Nat.0)), fin_succ(Nat.0.suc, fin_zero(Nat.0))) =
            (matrix2_det[F](Nat.0, a).inverse * (-matrix2_entry_10[F](Nat.0, a))) * matrix2_entry_01[F](Nat.0, a) +
            (matrix2_det[F](Nat.0, a).inverse * matrix2_entry_00[F](Nat.0, a)) * matrix2_entry_11[F](Nat.0, a)
        (matrix2_det[F](Nat.0, a).inverse * (-matrix2_entry_10[F](Nat.0, a))) * matrix2_entry_01[F](Nat.0, a) =
            matrix2_det[F](Nat.0, a).inverse * ((-matrix2_entry_10[F](Nat.0, a)) * matrix2_entry_01[F](Nat.0, a))
        (matrix2_det[F](Nat.0, a).inverse * matrix2_entry_00[F](Nat.0, a)) * matrix2_entry_11[F](Nat.0, a) =
            matrix2_det[F](Nat.0, a).inverse * (matrix2_entry_00[F](Nat.0, a) * matrix2_entry_11[F](Nat.0, a))
        (-matrix2_entry_10[F](Nat.0, a)) * matrix2_entry_01[F](Nat.0, a) =
            -(matrix2_entry_10[F](Nat.0, a) * matrix2_entry_01[F](Nat.0, a))
        matrix2_entry_10[F](Nat.0, a) * matrix2_entry_01[F](Nat.0, a) =
            matrix2_entry_01[F](Nat.0, a) * matrix2_entry_10[F](Nat.0, a)
        matrix_entry[F](Nat.0.suc.suc, Nat.0.suc.suc, square_matrix_mul[F](Nat.0.suc.suc, matrix2_inv[F](Nat.0, a), a), fin_succ(Nat.0.suc, fin_zero(Nat.0)), fin_succ(Nat.0.suc, fin_zero(Nat.0))) =
            matrix2_det[F](Nat.0, a).inverse *
                (-(matrix2_entry_10[F](Nat.0, a) * matrix2_entry_01[F](Nat.0, a)) +
                    matrix2_entry_00[F](Nat.0, a) * matrix2_entry_11[F](Nat.0, a))
        matrix2_det[F](Nat.0, a).inverse *
            (-(matrix2_entry_10[F](Nat.0, a) * matrix2_entry_01[F](Nat.0, a)) +
                matrix2_entry_00[F](Nat.0, a) * matrix2_entry_11[F](Nat.0, a)) =
            matrix2_det[F](Nat.0, a).inverse *
                (matrix2_entry_00[F](Nat.0, a) * matrix2_entry_11[F](Nat.0, a) -
                    matrix2_entry_01[F](Nat.0, a) * matrix2_entry_10[F](Nat.0, a))
        matrix_entry[F](Nat.0.suc.suc, Nat.0.suc.suc, square_matrix_mul[F](Nat.0.suc.suc, matrix2_inv[F](Nat.0, a), a), fin_succ(Nat.0.suc, fin_zero(Nat.0)), fin_succ(Nat.0.suc, fin_zero(Nat.0))) =
            matrix2_det[F](Nat.0, a).inverse *
                (matrix2_entry_00[F](Nat.0, a) * matrix2_entry_11[F](Nat.0, a) -
                    matrix2_entry_01[F](Nat.0, a) * matrix2_entry_10[F](Nat.0, a))
        matrix2_det_unfold[F](a)
        matrix_entry[F](Nat.0.suc.suc, Nat.0.suc.suc, square_matrix_mul[F](Nat.0.suc.suc, matrix2_inv[F](Nat.0, a), a), fin_succ(Nat.0.suc, fin_zero(Nat.0)), fin_succ(Nat.0.suc, fin_zero(Nat.0))) =
            matrix2_det[F](Nat.0, a).inverse * matrix2_det[F](Nat.0, a)
        mul_inverse_left[F](matrix2_det[F](Nat.0, a))
        matrix2_det[F](Nat.0, a).inverse * matrix2_det[F](Nat.0, a) = F.1
        matrix_entry[F](Nat.0.suc.suc, Nat.0.suc.suc, square_matrix_mul[F](Nat.0.suc.suc, matrix2_inv[F](Nat.0, a), a), fin_succ(Nat.0.suc, fin_zero(Nat.0)), fin_succ(Nat.0.suc, fin_zero(Nat.0))) = F.1
    }
}

/// Multiplying by the inverse on the left gives the identity, when the
/// determinant is nonzero.
theorem matrix2_inv_left[F: Field](a: Matrix[F, Nat.0.suc.suc, Nat.0.suc.suc]) {
    matrix2_det[F](Nat.0, a) != F.0 implies
        square_matrix_mul[F](Nat.0.suc.suc, matrix2_inv[F](Nat.0, a), a) = matrix_one[F](Nat.0.suc.suc)
} by {
    if matrix2_det[F](Nat.0, a) != F.0 {
        let lhs = square_matrix_mul[F](Nat.0.suc.suc, matrix2_inv[F](Nat.0, a), a)
        forall(i: Fin[Nat.0.suc.suc], j: Fin[Nat.0.suc.suc]) {
            if i.value = Nat.0 {
                if j.value = Nat.0 {
                    fin_two_zero_value_two
                    i.value = fin_zero(Nat.0.suc).value
                    ext(Nat.0.suc.suc, i, fin_zero(Nat.0.suc))
                    i = fin_zero(Nat.0.suc)
                    j.value = fin_zero(Nat.0.suc).value
                    ext(Nat.0.suc.suc, j, fin_zero(Nat.0.suc))
                    j = fin_zero(Nat.0.suc)
                    matrix2_det[F](Nat.0, a) != F.0
                    matrix2_inv_left_entry_00[F](a)
                    matrix_entry[F](Nat.0.suc.suc, Nat.0.suc.suc, square_matrix_mul[F](Nat.0.suc.suc, matrix2_inv[F](Nat.0, a), a), fin_zero(Nat.0.suc), fin_zero(Nat.0.suc)) = F.1
                    matrix_entry[F](Nat.0.suc.suc, Nat.0.suc.suc, lhs, fin_zero(Nat.0.suc), fin_zero(Nat.0.suc)) = F.1
                    matrix_entry[F](Nat.0.suc.suc, Nat.0.suc.suc, lhs, i, j) = F.1
                    j = i
                    matrix_entry_one_diag[F](Nat.0.suc.suc, i)
                    matrix_entry[F](Nat.0.suc.suc, Nat.0.suc.suc, matrix_one[F](Nat.0.suc.suc), i, j) = F.1
                    matrix_entry[F](Nat.0.suc.suc, Nat.0.suc.suc, lhs, i, j) =
                        matrix_entry[F](Nat.0.suc.suc, Nat.0.suc.suc, matrix_one[F](Nat.0.suc.suc), i, j)
                } else {
                    fin_two_zero_value_two
                    i.value = fin_zero(Nat.0.suc).value
                    ext(Nat.0.suc.suc, i, fin_zero(Nat.0.suc))
                    i = fin_zero(Nat.0.suc)
                    fin_two_one_value_two
                    j.value = fin_succ(Nat.0.suc, fin_zero(Nat.0)).value
                    ext(Nat.0.suc.suc, j, fin_succ(Nat.0.suc, fin_zero(Nat.0)))
                    j = fin_succ(Nat.0.suc, fin_zero(Nat.0))
                    matrix2_det[F](Nat.0, a) != F.0
                    matrix2_inv_left_entry_01[F](a)
                    matrix_entry[F](Nat.0.suc.suc, Nat.0.suc.suc, square_matrix_mul[F](Nat.0.suc.suc, matrix2_inv[F](Nat.0, a), a), fin_zero(Nat.0.suc), fin_succ(Nat.0.suc, fin_zero(Nat.0))) = F.0
                    matrix_entry[F](Nat.0.suc.suc, Nat.0.suc.suc, lhs, fin_zero(Nat.0.suc), fin_succ(Nat.0.suc, fin_zero(Nat.0))) = F.0
                    matrix_entry[F](Nat.0.suc.suc, Nat.0.suc.suc, lhs, i, j) = F.0
                    fin_two_zero_ne_one
                    i != j
                    matrix_entry_one_off_diag[F](Nat.0.suc.suc, i, j)
                    matrix_entry[F](Nat.0.suc.suc, Nat.0.suc.suc, matrix_one[F](Nat.0.suc.suc), i, j) = F.0
                    matrix_entry[F](Nat.0.suc.suc, Nat.0.suc.suc, lhs, i, j) =
                        matrix_entry[F](Nat.0.suc.suc, Nat.0.suc.suc, matrix_one[F](Nat.0.suc.suc), i, j)
                }
            } else {
                if j.value = Nat.0 {
                    fin_two_one_value_two
                    i.value = fin_succ(Nat.0.suc, fin_zero(Nat.0)).value
                    ext(Nat.0.suc.suc, i, fin_succ(Nat.0.suc, fin_zero(Nat.0)))
                    i = fin_succ(Nat.0.suc, fin_zero(Nat.0))
                    fin_two_zero_value_two
                    j.value = fin_zero(Nat.0.suc).value
                    ext(Nat.0.suc.suc, j, fin_zero(Nat.0.suc))
                    j = fin_zero(Nat.0.suc)
                    matrix2_det[F](Nat.0, a) != F.0
                    matrix2_inv_left_entry_10[F](a)
                    matrix_entry[F](Nat.0.suc.suc, Nat.0.suc.suc, square_matrix_mul[F](Nat.0.suc.suc, matrix2_inv[F](Nat.0, a), a), fin_succ(Nat.0.suc, fin_zero(Nat.0)), fin_zero(Nat.0.suc)) = F.0
                    matrix_entry[F](Nat.0.suc.suc, Nat.0.suc.suc, lhs, fin_succ(Nat.0.suc, fin_zero(Nat.0)), fin_zero(Nat.0.suc)) = F.0
                    matrix_entry[F](Nat.0.suc.suc, Nat.0.suc.suc, lhs, i, j) = F.0
                    fin_two_one_ne_zero
                    i != j
                    matrix_entry_one_off_diag[F](Nat.0.suc.suc, i, j)
                    matrix_entry[F](Nat.0.suc.suc, Nat.0.suc.suc, matrix_one[F](Nat.0.suc.suc), i, j) = F.0
                    matrix_entry[F](Nat.0.suc.suc, Nat.0.suc.suc, lhs, i, j) =
                        matrix_entry[F](Nat.0.suc.suc, Nat.0.suc.suc, matrix_one[F](Nat.0.suc.suc), i, j)
                } else {
                    fin_two_one_value_two
                    i.value = fin_succ(Nat.0.suc, fin_zero(Nat.0)).value
                    ext(Nat.0.suc.suc, i, fin_succ(Nat.0.suc, fin_zero(Nat.0)))
                    i = fin_succ(Nat.0.suc, fin_zero(Nat.0))
                    j.value = fin_succ(Nat.0.suc, fin_zero(Nat.0)).value
                    ext(Nat.0.suc.suc, j, fin_succ(Nat.0.suc, fin_zero(Nat.0)))
                    j = fin_succ(Nat.0.suc, fin_zero(Nat.0))
                    matrix2_det[F](Nat.0, a) != F.0
                    matrix2_inv_left_entry_11[F](a)
                    matrix_entry[F](Nat.0.suc.suc, Nat.0.suc.suc, square_matrix_mul[F](Nat.0.suc.suc, matrix2_inv[F](Nat.0, a), a), fin_succ(Nat.0.suc, fin_zero(Nat.0)), fin_succ(Nat.0.suc, fin_zero(Nat.0))) = F.1
                    matrix_entry[F](Nat.0.suc.suc, Nat.0.suc.suc, lhs, fin_succ(Nat.0.suc, fin_zero(Nat.0)), fin_succ(Nat.0.suc, fin_zero(Nat.0))) = F.1
                    matrix_entry[F](Nat.0.suc.suc, Nat.0.suc.suc, lhs, i, j) = F.1
                    j = i
                    matrix_entry_one_diag[F](Nat.0.suc.suc, i)
                    matrix_entry[F](Nat.0.suc.suc, Nat.0.suc.suc, matrix_one[F](Nat.0.suc.suc), i, j) = F.1
                    matrix_entry[F](Nat.0.suc.suc, Nat.0.suc.suc, lhs, i, j) =
                        matrix_entry[F](Nat.0.suc.suc, Nat.0.suc.suc, matrix_one[F](Nat.0.suc.suc), i, j)
                }
            }
            matrix_entry[F](Nat.0.suc.suc, Nat.0.suc.suc, lhs, i, j) =
                matrix_entry[F](Nat.0.suc.suc, Nat.0.suc.suc, matrix_one[F](Nat.0.suc.suc), i, j)
        }
        matrix_ext[F](Nat.0.suc.suc, Nat.0.suc.suc, lhs, matrix_one[F](Nat.0.suc.suc))
        lhs = matrix_one[F](Nat.0.suc.suc)
        square_matrix_mul[F](Nat.0.suc.suc, matrix2_inv[F](Nat.0, a), a) = matrix_one[F](Nat.0.suc.suc)
    }
}

// ---------------------------------------------------------------------------
// Uniqueness of the inverse
// ---------------------------------------------------------------------------

/// Any two two-sided inverses of the same matrix agree.
theorem square_matrix_inv_unique_two[F: Ring](
    a: Matrix[F, Nat.0.suc.suc, Nat.0.suc.suc],
    b: Matrix[F, Nat.0.suc.suc, Nat.0.suc.suc],
    c: Matrix[F, Nat.0.suc.suc, Nat.0.suc.suc]
) {
    (square_matrix_mul[F](Nat.0.suc.suc, a, b) = matrix_one[F](Nat.0.suc.suc) and
        square_matrix_mul[F](Nat.0.suc.suc, b, a) = matrix_one[F](Nat.0.suc.suc) and
        square_matrix_mul[F](Nat.0.suc.suc, a, c) = matrix_one[F](Nat.0.suc.suc) and
        square_matrix_mul[F](Nat.0.suc.suc, c, a) = matrix_one[F](Nat.0.suc.suc)) implies
        b = c
} by {
    if square_matrix_mul[F](Nat.0.suc.suc, a, b) = matrix_one[F](Nat.0.suc.suc) and
        square_matrix_mul[F](Nat.0.suc.suc, b, a) = matrix_one[F](Nat.0.suc.suc) and
        square_matrix_mul[F](Nat.0.suc.suc, a, c) = matrix_one[F](Nat.0.suc.suc) and
        square_matrix_mul[F](Nat.0.suc.suc, c, a) = matrix_one[F](Nat.0.suc.suc) {
        square_matrix_mul_one_right_two[F](b)
        square_matrix_mul[F](Nat.0.suc.suc, b, matrix_one[F](Nat.0.suc.suc)) = b
        square_matrix_mul[F](Nat.0.suc.suc, b, square_matrix_mul[F](Nat.0.suc.suc, a, c)) =
            square_matrix_mul[F](Nat.0.suc.suc, b, matrix_one[F](Nat.0.suc.suc))
        square_matrix_mul_assoc_two[F](b, a, c)
        square_matrix_mul[F](Nat.0.suc.suc, square_matrix_mul[F](Nat.0.suc.suc, b, a), c) =
            square_matrix_mul[F](Nat.0.suc.suc, b, square_matrix_mul[F](Nat.0.suc.suc, a, c))
        square_matrix_mul[F](Nat.0.suc.suc, square_matrix_mul[F](Nat.0.suc.suc, b, a), c) =
            square_matrix_mul[F](Nat.0.suc.suc, matrix_one[F](Nat.0.suc.suc), c)
        square_matrix_mul_one_left_two[F](c)
        square_matrix_mul[F](Nat.0.suc.suc, matrix_one[F](Nat.0.suc.suc), c) = c
        b = c
    }
}

/// A two-sided inverse of a matrix with nonzero determinant is its inverse.
theorem matrix2_inv_unique[F: Field](
    a: Matrix[F, Nat.0.suc.suc, Nat.0.suc.suc], b: Matrix[F, Nat.0.suc.suc, Nat.0.suc.suc]
) {
    matrix2_det[F](Nat.0, a) != F.0 implies
        ((square_matrix_mul[F](Nat.0.suc.suc, a, b) = matrix_one[F](Nat.0.suc.suc) and
            square_matrix_mul[F](Nat.0.suc.suc, b, a) = matrix_one[F](Nat.0.suc.suc)) implies
            b = matrix2_inv[F](Nat.0, a))
} by {
    if matrix2_det[F](Nat.0, a) != F.0 {
        if square_matrix_mul[F](Nat.0.suc.suc, a, b) = matrix_one[F](Nat.0.suc.suc) and
            square_matrix_mul[F](Nat.0.suc.suc, b, a) = matrix_one[F](Nat.0.suc.suc) {
            matrix2_det[F](Nat.0, a) != F.0
            matrix2_inv_right[F](a)
            square_matrix_mul[F](Nat.0.suc.suc, a, matrix2_inv[F](Nat.0, a)) = matrix_one[F](Nat.0.suc.suc)
            matrix2_inv_left[F](a)
            square_matrix_mul[F](Nat.0.suc.suc, matrix2_inv[F](Nat.0, a), a) = matrix_one[F](Nat.0.suc.suc)
            square_matrix_inv_unique_two[F](a, b, matrix2_inv[F](Nat.0, a))
            b = matrix2_inv[F](Nat.0, a)
        }
    }
}

/// A right inverse of a matrix with nonzero determinant is its inverse.
theorem matrix2_inv_unique_right[F: Field](
    a: Matrix[F, Nat.0.suc.suc, Nat.0.suc.suc], b: Matrix[F, Nat.0.suc.suc, Nat.0.suc.suc]
) {
    matrix2_det[F](Nat.0, a) != F.0 implies
        ((square_matrix_mul[F](Nat.0.suc.suc, a, b) = matrix_one[F](Nat.0.suc.suc)) implies
            b = matrix2_inv[F](Nat.0, a))
} by {
    if matrix2_det[F](Nat.0, a) != F.0 {
        if square_matrix_mul[F](Nat.0.suc.suc, a, b) = matrix_one[F](Nat.0.suc.suc) {
            square_matrix_mul_one_left_two[F](b)
            square_matrix_mul[F](Nat.0.suc.suc, matrix_one[F](Nat.0.suc.suc), b) = b
            matrix2_inv_left[F](a)
            square_matrix_mul[F](Nat.0.suc.suc, matrix2_inv[F](Nat.0, a), a) = matrix_one[F](Nat.0.suc.suc)
            square_matrix_mul[F](Nat.0.suc.suc, matrix_one[F](Nat.0.suc.suc), b) =
                square_matrix_mul[F](Nat.0.suc.suc, square_matrix_mul[F](Nat.0.suc.suc, matrix2_inv[F](Nat.0, a), a), b)
            square_matrix_mul_assoc_two[F](matrix2_inv[F](Nat.0, a), a, b)
            square_matrix_mul[F](Nat.0.suc.suc, square_matrix_mul[F](Nat.0.suc.suc, matrix2_inv[F](Nat.0, a), a), b) =
                square_matrix_mul[F](Nat.0.suc.suc, matrix2_inv[F](Nat.0, a), square_matrix_mul[F](Nat.0.suc.suc, a, b))
            square_matrix_mul[F](Nat.0.suc.suc, matrix2_inv[F](Nat.0, a), square_matrix_mul[F](Nat.0.suc.suc, a, b)) =
                square_matrix_mul[F](Nat.0.suc.suc, matrix2_inv[F](Nat.0, a), matrix_one[F](Nat.0.suc.suc))
            square_matrix_mul_one_right_two[F](matrix2_inv[F](Nat.0, a))
            square_matrix_mul[F](Nat.0.suc.suc, matrix2_inv[F](Nat.0, a), matrix_one[F](Nat.0.suc.suc)) =
                matrix2_inv[F](Nat.0, a)
            b = matrix2_inv[F](Nat.0, a)
        }
    }
}

// ---------------------------------------------------------------------------
// The determinant of a product
// ---------------------------------------------------------------------------

/// Swapping the middle two factors of a four-factor product.
theorem ring4_mul_swap_second[F: CommRing](a: F, b: F, c: F, d: F) {
    a * b * c * d = a * c * b * d
} by {
    a * b * c * d = (a * (b * c)) * d
    b * c = c * b
    (a * (b * c)) * d = (a * (c * b)) * d
    (a * (c * b)) * d = a * c * b * d
}

/// Swapping the last two factors of a four-factor product.
theorem ring4_mul_swap_fourth[F: CommRing](a: F, b: F, c: F, d: F) {
    a * b * c * d = a * b * d * c
} by {
    a * b * c * d = (a * b) * (c * d)
    c * d = d * c
    (a * b) * (c * d) = (a * b) * (d * c)
    (a * b) * (d * c) = a * b * d * c
}

/// Cancelling a common leading summand in a difference of sums.
theorem ring_sub_cancel_common[F: Ring](a: F, b: F, c: F) {
    (a + b) - (a + c) = b - c
} by {
    (a + b) - (a + c) = (a + b) + -(a + c)
    -(a + c) = -c + -a
    (a + b) + -(a + c) = (a + b) + (-c + -a)
    (a + b) + (-c + -a) = (a + -a) + (b + -c)
    (a + -a) + (b + -c) = b + -c
    b + -c = b - c
    (a + b) - (a + c) = b - c
}

/// Subtracting a difference is subtracting then adding.
theorem ring_sub_sub[F: Ring](a: F, b: F, c: F, d: F) {
    (a - b) - (c - d) = a - b - c + d
} by {
    (a - b) - (c - d) = (a + -b) + -(c + -d)
    -(c + -d) = -(-d) + -c
    -(-d) = d
    -(c + -d) = d + -c
    (a + -b) + -(c + -d) = (a + -b) + (d + -c)
    (a + -b) + (d + -c) = a + -b - c + d
    (a - b) - (c - d) = a - b - c + d
}

/// Moving a leading added term to the back of a difference chain.
theorem ring_sub_add_shift[F: Ring](a: F, b: F, c: F, d: F) {
    a + b - c - d = a - c - d + b
} by {
    a + b - c - d = (a + b) + -c + -d
    (a + b) + -c + -d = (a + -c) + b + -d
    (a + -c) + b + -d = (a + -c + -d) + b
    (a + -c + -d) + b = a - c - d + b
    a + b - c - d = a - c - d + b
}

/// Swapping the two subtracted middle terms of a four-term difference chain.
theorem ring4_sub_swap[F: Ring](a: F, b: F, c: F, d: F) {
    a - b - c + d = a - c - b + d
} by {
    a - b - c + d = (a + -b) + (-c + d)
    add_swap_inner[F](a, -b, -c, d)
    (a + -b) + (-c + d) = (a + -c) + (-b + d)
    (a + -c) + (-b + d) = a - c - b + d
    a - b - c + d = a - c - b + d
}

/// Cancelling matching outer terms in a difference of four-term sums.
theorem ring4_sub_cancel[F: Ring](a: F, b: F, c: F, d: F, e: F, f: F) {
    (a + b + c + d) - (a + e + f + d) = b + c - e - f
} by {
    (a + b + c + d) - (a + e + f + d) = (a + (b + c) + d) - (a + (e + f) + d)
    ring_sub_cancel_common[F](a + d, b + c, e + f)
    (a + (b + c) + d) - (a + (e + f) + d) = (b + c) - (e + f)
    (b + c) - (e + f) = (b + c) + -(e + f)
    -(e + f) = -f + -e
    (b + c) + -(e + f) = (b + c) + (-f + -e)
    (b + c) + (-f + -e) = b + c + (-e + -f)
    b + c + (-e + -f) = b + c - e - f
    (b + c) - (e + f) = b + c - e - f
    (a + b + c + d) - (a + e + f + d) = b + c - e - f
}

/// A difference of products distributes into four signed products.
theorem ring_sub_mul_sub[F: Ring](a: F, b: F, c: F, d: F) {
    (a - b) * (c - d) = a * c - a * d - b * c + b * d
} by {
    (a - b) * (c - d) = (a - b) * c - (a - b) * d
    (a - b) * c = a * c - b * c
    (a - b) * d = a * d - b * d
    (a - b) * (c - d) = (a * c - b * c) - (a * d - b * d)
    ring_sub_sub[F](a * c, b * c, a * d, b * d)
    (a * c - b * c) - (a * d - b * d) = a * c - b * c - a * d + b * d
    ring4_sub_swap[F](a * c, b * c, a * d, b * d)
    a * c - b * c - a * d + b * d = a * c - a * d - b * c + b * d
    (a - b) * (c - d) = a * c - a * d - b * c + b * d
}

/// The determinant of a product of two-by-two entry blocks, expanded.
theorem det2_entry_mul_scalars[F: CommRing](
    a00: F, a01: F, a10: F, a11: F, b00: F, b01: F, b10: F, b11: F
) {
    det2_entry[F](a00 * b00 + a01 * b10, a00 * b01 + a01 * b11, a10 * b00 + a11 * b10, a10 * b01 + a11 * b11) =
        det2_entry[F](a00, a01, a10, a11) * det2_entry[F](b00, b01, b10, b11)
} by {
    det2_entry_expanded[F](a00 * b00 + a01 * b10, a00 * b01 + a01 * b11, a10 * b00 + a11 * b10, a10 * b01 + a11 * b11)
    det2_entry[F](a00 * b00 + a01 * b10, a00 * b01 + a01 * b11, a10 * b00 + a11 * b10, a10 * b01 + a11 * b11) =
        (a00 * b00 + a01 * b10) * (a10 * b01 + a11 * b11) -
        (a00 * b01 + a01 * b11) * (a10 * b00 + a11 * b10)
    (a00 * b00 + a01 * b10) * (a10 * b01 + a11 * b11) =
        a00 * b00 * (a10 * b01 + a11 * b11) + a01 * b10 * (a10 * b01 + a11 * b11)
    a00 * b00 * (a10 * b01 + a11 * b11) = a00 * b00 * a10 * b01 + a00 * b00 * a11 * b11
    a01 * b10 * (a10 * b01 + a11 * b11) = a01 * b10 * a10 * b01 + a01 * b10 * a11 * b11
    (a00 * b00 + a01 * b10) * (a10 * b01 + a11 * b11) =
        a00 * b00 * a10 * b01 + a00 * b00 * a11 * b11 + a01 * b10 * a10 * b01 + a01 * b10 * a11 * b11
    (a00 * b01 + a01 * b11) * (a10 * b00 + a11 * b10) =
        a00 * b01 * (a10 * b00 + a11 * b10) + a01 * b11 * (a10 * b00 + a11 * b10)
    a00 * b01 * (a10 * b00 + a11 * b10) = a00 * b01 * a10 * b00 + a00 * b01 * a11 * b10
    a01 * b11 * (a10 * b00 + a11 * b10) = a01 * b11 * a10 * b00 + a01 * b11 * a11 * b10
    (a00 * b01 + a01 * b11) * (a10 * b00 + a11 * b10) =
        a00 * b01 * a10 * b00 + a00 * b01 * a11 * b10 + a01 * b11 * a10 * b00 + a01 * b11 * a11 * b10
    ring4_mul_swap_second[F](a00, b00, a10, b01)
    ring4_mul_swap_second[F](a00, b00, a11, b11)
    ring4_mul_swap_second[F](a01, b10, a10, b01)
    ring4_mul_swap_second[F](a01, b10, a11, b11)
    ring4_mul_swap_second[F](a00, b01, a10, b00)
    ring4_mul_swap_fourth[F](a00, a10, b01, b00)
    ring4_mul_swap_second[F](a00, b01, a11, b10)
    ring4_mul_swap_second[F](a01, b11, a10, b00)
    ring4_mul_swap_fourth[F](a01, a10, b11, b00)
    ring4_mul_swap_second[F](a01, b11, a11, b10)
    (a00 * b00 + a01 * b10) * (a10 * b01 + a11 * b11) =
        a00 * a10 * b00 * b01 + a00 * a11 * b00 * b11 + a01 * a10 * b01 * b10 + a01 * a11 * b10 * b11
    (a00 * b01 + a01 * b11) * (a10 * b00 + a11 * b10) =
        a00 * a10 * b00 * b01 + a00 * a11 * b01 * b10 + a01 * a10 * b00 * b11 + a01 * a11 * b10 * b11
    ring4_sub_cancel[F](a00 * a10 * b00 * b01, a00 * a11 * b00 * b11, a01 * a10 * b01 * b10, a01 * a11 * b10 * b11, a00 * a11 * b01 * b10, a01 * a10 * b00 * b11)
    (a00 * a10 * b00 * b01 + a00 * a11 * b00 * b11 + a01 * a10 * b01 * b10 + a01 * a11 * b10 * b11) -
        (a00 * a10 * b00 * b01 + a00 * a11 * b01 * b10 + a01 * a10 * b00 * b11 + a01 * a11 * b10 * b11) =
        a00 * a11 * b00 * b11 + a01 * a10 * b01 * b10 - a00 * a11 * b01 * b10 - a01 * a10 * b00 * b11
    det2_entry[F](a00 * b00 + a01 * b10, a00 * b01 + a01 * b11, a10 * b00 + a11 * b10, a10 * b01 + a11 * b11) =
        a00 * a11 * b00 * b11 + a01 * a10 * b01 * b10 - a00 * a11 * b01 * b10 - a01 * a10 * b00 * b11
    det2_entry_expanded[F](a00, a01, a10, a11)
    det2_entry_expanded[F](b00, b01, b10, b11)
    det2_entry[F](a00, a01, a10, a11) * det2_entry[F](b00, b01, b10, b11) =
        (a00 * a11 - a01 * a10) * (b00 * b11 - b01 * b10)
    ring_sub_mul_sub[F](a00 * a11, a01 * a10, b00 * b11, b01 * b10)
    (a00 * a11 - a01 * a10) * (b00 * b11 - b01 * b10) =
        a00 * a11 * b00 * b11 - a00 * a11 * b01 * b10 - a01 * a10 * b00 * b11 + a01 * a10 * b01 * b10
    ring_sub_add_shift[F](a00 * a11 * b00 * b11, a01 * a10 * b01 * b10, a00 * a11 * b01 * b10, a01 * a10 * b00 * b11)
    a00 * a11 * b00 * b11 - a00 * a11 * b01 * b10 - a01 * a10 * b00 * b11 + a01 * a10 * b01 * b10 =
        a00 * a11 * b00 * b11 + a01 * a10 * b01 * b10 - a00 * a11 * b01 * b10 - a01 * a10 * b00 * b11
    det2_entry[F](a00, a01, a10, a11) * det2_entry[F](b00, b01, b10, b11) =
        a00 * a11 * b00 * b11 + a01 * a10 * b01 * b10 - a00 * a11 * b01 * b10 - a01 * a10 * b00 * b11
    det2_entry[F](a00 * b00 + a01 * b10, a00 * b01 + a01 * b11, a10 * b00 + a11 * b10, a10 * b01 + a11 * b11) =
        det2_entry[F](a00, a01, a10, a11) * det2_entry[F](b00, b01, b10, b11)
}

/// The determinant of the two-by-two identity matrix is one.
theorem matrix2_det_one[F: Field] {
    matrix2_det[F](Nat.0, matrix_one[F](Nat.0.suc.suc)) = F.1
} by {
    matrix2_det_unfold[F](matrix_one[F](Nat.0.suc.suc))
    matrix2_det[F](Nat.0, matrix_one[F](Nat.0.suc.suc)) =
        matrix2_entry_00[F](Nat.0, matrix_one[F](Nat.0.suc.suc)) * matrix2_entry_11[F](Nat.0, matrix_one[F](Nat.0.suc.suc)) -
        matrix2_entry_01[F](Nat.0, matrix_one[F](Nat.0.suc.suc)) * matrix2_entry_10[F](Nat.0, matrix_one[F](Nat.0.suc.suc))
    matrix_entry_one_diag[F](Nat.0.suc.suc, fin_zero(Nat.0.suc))
    matrix_entry_one_diag[F](Nat.0.suc.suc, fin_succ(Nat.0.suc, fin_zero(Nat.0)))
    fin_two_zero_ne_one
    fin_two_one_ne_zero
    matrix_entry_one_off_diag[F](Nat.0.suc.suc, fin_zero(Nat.0.suc), fin_succ(Nat.0.suc, fin_zero(Nat.0)))
    matrix_entry_one_off_diag[F](Nat.0.suc.suc, fin_succ(Nat.0.suc, fin_zero(Nat.0)), fin_zero(Nat.0.suc))
    matrix2_entry_00[F](Nat.0, matrix_one[F](Nat.0.suc.suc)) =
        matrix_entry[F](Nat.0.suc.suc, Nat.0.suc.suc, matrix_one[F](Nat.0.suc.suc), fin_zero(Nat.0.suc), fin_zero(Nat.0.suc))
    matrix2_entry_11[F](Nat.0, matrix_one[F](Nat.0.suc.suc)) =
        matrix_entry[F](Nat.0.suc.suc, Nat.0.suc.suc, matrix_one[F](Nat.0.suc.suc), fin_succ(Nat.0.suc, fin_zero(Nat.0)), fin_succ(Nat.0.suc, fin_zero(Nat.0)))
    matrix2_entry_01[F](Nat.0, matrix_one[F](Nat.0.suc.suc)) =
        matrix_entry[F](Nat.0.suc.suc, Nat.0.suc.suc, matrix_one[F](Nat.0.suc.suc), fin_zero(Nat.0.suc), fin_succ(Nat.0.suc, fin_zero(Nat.0)))
    matrix2_entry_10[F](Nat.0, matrix_one[F](Nat.0.suc.suc)) =
        matrix_entry[F](Nat.0.suc.suc, Nat.0.suc.suc, matrix_one[F](Nat.0.suc.suc), fin_succ(Nat.0.suc, fin_zero(Nat.0)), fin_zero(Nat.0.suc))
    matrix2_entry_00[F](Nat.0, matrix_one[F](Nat.0.suc.suc)) = F.1
    matrix2_entry_11[F](Nat.0, matrix_one[F](Nat.0.suc.suc)) = F.1
    matrix2_entry_01[F](Nat.0, matrix_one[F](Nat.0.suc.suc)) = F.0
    matrix2_entry_10[F](Nat.0, matrix_one[F](Nat.0.suc.suc)) = F.0
    matrix2_det[F](Nat.0, matrix_one[F](Nat.0.suc.suc)) = F.1 * F.1 - F.0 * F.0
    F.1 * F.1 = F.1
    F.0 * F.0 = F.0
    F.1 - F.0 = F.1
    matrix2_det[F](Nat.0, matrix_one[F](Nat.0.suc.suc)) = F.1
}

/// The determinant is multiplicative for two-by-two matrices.
theorem matrix2_det_mul[F: Field](
    a: Matrix[F, Nat.0.suc.suc, Nat.0.suc.suc], b: Matrix[F, Nat.0.suc.suc, Nat.0.suc.suc]
) {
    matrix2_det[F](Nat.0, square_matrix_mul[F](Nat.0.suc.suc, a, b)) =
        matrix2_det[F](Nat.0, a) * matrix2_det[F](Nat.0, b)
} by {
    matrix2_det_unfold[F](square_matrix_mul[F](Nat.0.suc.suc, a, b))
    matrix2_det[F](Nat.0, square_matrix_mul[F](Nat.0.suc.suc, a, b)) =
        matrix2_entry_00[F](Nat.0, square_matrix_mul[F](Nat.0.suc.suc, a, b)) * matrix2_entry_11[F](Nat.0, square_matrix_mul[F](Nat.0.suc.suc, a, b)) -
        matrix2_entry_01[F](Nat.0, square_matrix_mul[F](Nat.0.suc.suc, a, b)) * matrix2_entry_10[F](Nat.0, square_matrix_mul[F](Nat.0.suc.suc, a, b))
    matrix2_mul_entry_00[F](a, b)
    matrix2_mul_entry_11[F](a, b)
    matrix2_mul_entry_01[F](a, b)
    matrix2_mul_entry_10[F](a, b)
    matrix2_det[F](Nat.0, square_matrix_mul[F](Nat.0.suc.suc, a, b)) =
        (matrix2_entry_00[F](Nat.0, a) * matrix2_entry_00[F](Nat.0, b) + matrix2_entry_01[F](Nat.0, a) * matrix2_entry_10[F](Nat.0, b)) *
            (matrix2_entry_10[F](Nat.0, a) * matrix2_entry_01[F](Nat.0, b) + matrix2_entry_11[F](Nat.0, a) * matrix2_entry_11[F](Nat.0, b)) -
        (matrix2_entry_00[F](Nat.0, a) * matrix2_entry_01[F](Nat.0, b) + matrix2_entry_01[F](Nat.0, a) * matrix2_entry_11[F](Nat.0, b)) *
            (matrix2_entry_10[F](Nat.0, a) * matrix2_entry_00[F](Nat.0, b) + matrix2_entry_11[F](Nat.0, a) * matrix2_entry_10[F](Nat.0, b))
    det2_entry_expanded[F](
        matrix2_entry_00[F](Nat.0, a) * matrix2_entry_00[F](Nat.0, b) + matrix2_entry_01[F](Nat.0, a) * matrix2_entry_10[F](Nat.0, b),
        matrix2_entry_00[F](Nat.0, a) * matrix2_entry_01[F](Nat.0, b) + matrix2_entry_01[F](Nat.0, a) * matrix2_entry_11[F](Nat.0, b),
        matrix2_entry_10[F](Nat.0, a) * matrix2_entry_00[F](Nat.0, b) + matrix2_entry_11[F](Nat.0, a) * matrix2_entry_10[F](Nat.0, b),
        matrix2_entry_10[F](Nat.0, a) * matrix2_entry_01[F](Nat.0, b) + matrix2_entry_11[F](Nat.0, a) * matrix2_entry_11[F](Nat.0, b)
    )
    matrix2_det[F](Nat.0, square_matrix_mul[F](Nat.0.suc.suc, a, b)) =
        det2_entry[F](
            matrix2_entry_00[F](Nat.0, a) * matrix2_entry_00[F](Nat.0, b) + matrix2_entry_01[F](Nat.0, a) * matrix2_entry_10[F](Nat.0, b),
            matrix2_entry_00[F](Nat.0, a) * matrix2_entry_01[F](Nat.0, b) + matrix2_entry_01[F](Nat.0, a) * matrix2_entry_11[F](Nat.0, b),
            matrix2_entry_10[F](Nat.0, a) * matrix2_entry_00[F](Nat.0, b) + matrix2_entry_11[F](Nat.0, a) * matrix2_entry_10[F](Nat.0, b),
            matrix2_entry_10[F](Nat.0, a) * matrix2_entry_01[F](Nat.0, b) + matrix2_entry_11[F](Nat.0, a) * matrix2_entry_11[F](Nat.0, b)
        )
    det2_entry_mul_scalars[F](
        matrix2_entry_00[F](Nat.0, a), matrix2_entry_01[F](Nat.0, a), matrix2_entry_10[F](Nat.0, a), matrix2_entry_11[F](Nat.0, a),
        matrix2_entry_00[F](Nat.0, b), matrix2_entry_01[F](Nat.0, b), matrix2_entry_10[F](Nat.0, b), matrix2_entry_11[F](Nat.0, b)
    )
    matrix2_det[F](Nat.0, square_matrix_mul[F](Nat.0.suc.suc, a, b)) =
        det2_entry[F](matrix2_entry_00[F](Nat.0, a), matrix2_entry_01[F](Nat.0, a), matrix2_entry_10[F](Nat.0, a), matrix2_entry_11[F](Nat.0, a)) *
        det2_entry[F](matrix2_entry_00[F](Nat.0, b), matrix2_entry_01[F](Nat.0, b), matrix2_entry_10[F](Nat.0, b), matrix2_entry_11[F](Nat.0, b))
    matrix2_det_unfold[F](a)
    matrix2_det_unfold[F](b)
    matrix2_det[F](Nat.0, square_matrix_mul[F](Nat.0.suc.suc, a, b)) =
        matrix2_det[F](Nat.0, a) * matrix2_det[F](Nat.0, b)
}

// ---------------------------------------------------------------------------
// The inverse of a product
// ---------------------------------------------------------------------------

/// The inverse of a product is the product of the inverses in reverse order.
theorem matrix2_inv_mul[F: Field](
    a: Matrix[F, Nat.0.suc.suc, Nat.0.suc.suc], b: Matrix[F, Nat.0.suc.suc, Nat.0.suc.suc]
) {
    matrix2_det[F](Nat.0, a) != F.0 and matrix2_det[F](Nat.0, b) != F.0 implies
        matrix2_inv[F](Nat.0, square_matrix_mul[F](Nat.0.suc.suc, a, b)) =
            square_matrix_mul[F](Nat.0.suc.suc, matrix2_inv[F](Nat.0, b), matrix2_inv[F](Nat.0, a))
} by {
    if matrix2_det[F](Nat.0, a) != F.0 and matrix2_det[F](Nat.0, b) != F.0 {
        matrix2_det_mul[F](a, b)
        matrix2_det[F](Nat.0, square_matrix_mul[F](Nat.0.suc.suc, a, b)) =
            matrix2_det[F](Nat.0, a) * matrix2_det[F](Nat.0, b)
        matrix2_det[F](Nat.0, a) != F.0 and matrix2_det[F](Nat.0, b) != F.0
        field_mul_nonzero[F](matrix2_det[F](Nat.0, a), matrix2_det[F](Nat.0, b))
        matrix2_det[F](Nat.0, a) * matrix2_det[F](Nat.0, b) != F.0
        matrix2_det[F](Nat.0, square_matrix_mul[F](Nat.0.suc.suc, a, b)) != F.0
        matrix2_inv_right[F](square_matrix_mul[F](Nat.0.suc.suc, a, b))
        matrix2_inv_left[F](square_matrix_mul[F](Nat.0.suc.suc, a, b))
        square_matrix_mul[F](Nat.0.suc.suc, square_matrix_mul[F](Nat.0.suc.suc, a, b), matrix2_inv[F](Nat.0, square_matrix_mul[F](Nat.0.suc.suc, a, b))) =
            matrix_one[F](Nat.0.suc.suc)
        square_matrix_mul[F](Nat.0.suc.suc, matrix2_inv[F](Nat.0, square_matrix_mul[F](Nat.0.suc.suc, a, b)), square_matrix_mul[F](Nat.0.suc.suc, a, b)) =
            matrix_one[F](Nat.0.suc.suc)
        square_matrix_mul_assoc_two[F](a, b, square_matrix_mul[F](Nat.0.suc.suc, matrix2_inv[F](Nat.0, b), matrix2_inv[F](Nat.0, a)))
        square_matrix_mul[F](Nat.0.suc.suc, square_matrix_mul[F](Nat.0.suc.suc, a, b), square_matrix_mul[F](Nat.0.suc.suc, matrix2_inv[F](Nat.0, b), matrix2_inv[F](Nat.0, a))) =
            square_matrix_mul[F](Nat.0.suc.suc, a, square_matrix_mul[F](Nat.0.suc.suc, b, square_matrix_mul[F](Nat.0.suc.suc, matrix2_inv[F](Nat.0, b), matrix2_inv[F](Nat.0, a))))
        square_matrix_mul_assoc_two[F](b, matrix2_inv[F](Nat.0, b), matrix2_inv[F](Nat.0, a))
        square_matrix_mul[F](Nat.0.suc.suc, b, square_matrix_mul[F](Nat.0.suc.suc, matrix2_inv[F](Nat.0, b), matrix2_inv[F](Nat.0, a))) =
            square_matrix_mul[F](Nat.0.suc.suc, square_matrix_mul[F](Nat.0.suc.suc, b, matrix2_inv[F](Nat.0, b)), matrix2_inv[F](Nat.0, a))
        matrix2_det[F](Nat.0, b) != F.0
        matrix2_inv_right[F](b)
        square_matrix_mul[F](Nat.0.suc.suc, b, matrix2_inv[F](Nat.0, b)) = matrix_one[F](Nat.0.suc.suc)
        square_matrix_mul[F](Nat.0.suc.suc, b, square_matrix_mul[F](Nat.0.suc.suc, matrix2_inv[F](Nat.0, b), matrix2_inv[F](Nat.0, a))) =
            square_matrix_mul[F](Nat.0.suc.suc, matrix_one[F](Nat.0.suc.suc), matrix2_inv[F](Nat.0, a))
        square_matrix_mul_one_left_two[F](matrix2_inv[F](Nat.0, a))
        square_matrix_mul[F](Nat.0.suc.suc, matrix_one[F](Nat.0.suc.suc), matrix2_inv[F](Nat.0, a)) =
            matrix2_inv[F](Nat.0, a)
        square_matrix_mul[F](Nat.0.suc.suc, b, square_matrix_mul[F](Nat.0.suc.suc, matrix2_inv[F](Nat.0, b), matrix2_inv[F](Nat.0, a))) =
            matrix2_inv[F](Nat.0, a)
        square_matrix_mul[F](Nat.0.suc.suc, square_matrix_mul[F](Nat.0.suc.suc, a, b), square_matrix_mul[F](Nat.0.suc.suc, matrix2_inv[F](Nat.0, b), matrix2_inv[F](Nat.0, a))) =
            square_matrix_mul[F](Nat.0.suc.suc, a, matrix2_inv[F](Nat.0, a))
        matrix2_det[F](Nat.0, a) != F.0
        matrix2_inv_right[F](a)
        square_matrix_mul[F](Nat.0.suc.suc, a, matrix2_inv[F](Nat.0, a)) = matrix_one[F](Nat.0.suc.suc)
        square_matrix_mul[F](Nat.0.suc.suc, square_matrix_mul[F](Nat.0.suc.suc, a, b), square_matrix_mul[F](Nat.0.suc.suc, matrix2_inv[F](Nat.0, b), matrix2_inv[F](Nat.0, a))) =
            matrix_one[F](Nat.0.suc.suc)
        square_matrix_mul_assoc_two[F](matrix2_inv[F](Nat.0, b), matrix2_inv[F](Nat.0, a), b)
        square_matrix_mul[F](Nat.0.suc.suc, square_matrix_mul[F](Nat.0.suc.suc, matrix2_inv[F](Nat.0, b), matrix2_inv[F](Nat.0, a)), square_matrix_mul[F](Nat.0.suc.suc, a, b)) =
            square_matrix_mul[F](Nat.0.suc.suc, matrix2_inv[F](Nat.0, b), square_matrix_mul[F](Nat.0.suc.suc, matrix2_inv[F](Nat.0, a), square_matrix_mul[F](Nat.0.suc.suc, a, b)))
        square_matrix_mul_assoc_two[F](matrix2_inv[F](Nat.0, a), a, b)
        square_matrix_mul[F](Nat.0.suc.suc, matrix2_inv[F](Nat.0, a), square_matrix_mul[F](Nat.0.suc.suc, a, b)) =
            square_matrix_mul[F](Nat.0.suc.suc, square_matrix_mul[F](Nat.0.suc.suc, matrix2_inv[F](Nat.0, a), a), b)
        matrix2_inv_left[F](a)
        square_matrix_mul[F](Nat.0.suc.suc, matrix2_inv[F](Nat.0, a), a) = matrix_one[F](Nat.0.suc.suc)
        square_matrix_mul[F](Nat.0.suc.suc, matrix2_inv[F](Nat.0, a), square_matrix_mul[F](Nat.0.suc.suc, a, b)) =
            square_matrix_mul[F](Nat.0.suc.suc, matrix_one[F](Nat.0.suc.suc), b)
        square_matrix_mul_one_left_two[F](b)
        square_matrix_mul[F](Nat.0.suc.suc, matrix_one[F](Nat.0.suc.suc), b) = b
        square_matrix_mul[F](Nat.0.suc.suc, matrix2_inv[F](Nat.0, a), square_matrix_mul[F](Nat.0.suc.suc, a, b)) = b
        square_matrix_mul[F](Nat.0.suc.suc, square_matrix_mul[F](Nat.0.suc.suc, matrix2_inv[F](Nat.0, b), matrix2_inv[F](Nat.0, a)), square_matrix_mul[F](Nat.0.suc.suc, a, b)) =
            square_matrix_mul[F](Nat.0.suc.suc, matrix2_inv[F](Nat.0, b), b)
        matrix2_inv_left[F](b)
        square_matrix_mul[F](Nat.0.suc.suc, matrix2_inv[F](Nat.0, b), b) = matrix_one[F](Nat.0.suc.suc)
        square_matrix_mul[F](Nat.0.suc.suc, square_matrix_mul[F](Nat.0.suc.suc, matrix2_inv[F](Nat.0, b), matrix2_inv[F](Nat.0, a)), square_matrix_mul[F](Nat.0.suc.suc, a, b)) =
            matrix_one[F](Nat.0.suc.suc)
        square_matrix_inv_unique_two[F](square_matrix_mul[F](Nat.0.suc.suc, a, b), square_matrix_mul[F](Nat.0.suc.suc, matrix2_inv[F](Nat.0, b), matrix2_inv[F](Nat.0, a)), matrix2_inv[F](Nat.0, square_matrix_mul[F](Nat.0.suc.suc, a, b)))
        square_matrix_mul[F](Nat.0.suc.suc, matrix2_inv[F](Nat.0, b), matrix2_inv[F](Nat.0, a)) =
            matrix2_inv[F](Nat.0, square_matrix_mul[F](Nat.0.suc.suc, a, b))
        matrix2_inv[F](Nat.0, square_matrix_mul[F](Nat.0.suc.suc, a, b)) =
            square_matrix_mul[F](Nat.0.suc.suc, matrix2_inv[F](Nat.0, b), matrix2_inv[F](Nat.0, a))
    }
}

// ---------------------------------------------------------------------------
// The invertibility criterion
// ---------------------------------------------------------------------------

/// An invertible matrix has nonzero determinant.
theorem matrix2_invertible_implies_det_nonzero[F: Field](a: Matrix[F, Nat.0.suc.suc, Nat.0.suc.suc]) {
    (exists(b: Matrix[F, Nat.0.suc.suc, Nat.0.suc.suc]) {
        square_matrix_mul[F](Nat.0.suc.suc, a, b) = matrix_one[F](Nat.0.suc.suc) and
            square_matrix_mul[F](Nat.0.suc.suc, b, a) = matrix_one[F](Nat.0.suc.suc)
    }) implies matrix2_det[F](Nat.0, a) != F.0
} by {
    if exists(b: Matrix[F, Nat.0.suc.suc, Nat.0.suc.suc]) {
        square_matrix_mul[F](Nat.0.suc.suc, a, b) = matrix_one[F](Nat.0.suc.suc) and
            square_matrix_mul[F](Nat.0.suc.suc, b, a) = matrix_one[F](Nat.0.suc.suc)
    } {
        let b: Matrix[F, Nat.0.suc.suc, Nat.0.suc.suc] satisfy {
            square_matrix_mul[F](Nat.0.suc.suc, a, b) = matrix_one[F](Nat.0.suc.suc) and
                square_matrix_mul[F](Nat.0.suc.suc, b, a) = matrix_one[F](Nat.0.suc.suc)
        }
        matrix2_det_mul[F](a, b)
        matrix2_det[F](Nat.0, square_matrix_mul[F](Nat.0.suc.suc, a, b)) =
            matrix2_det[F](Nat.0, a) * matrix2_det[F](Nat.0, b)
        matrix2_det_one[F]
        matrix2_det[F](Nat.0, square_matrix_mul[F](Nat.0.suc.suc, a, b)) = F.1
        matrix2_det[F](Nat.0, a) * matrix2_det[F](Nat.0, b) = F.1
        if matrix2_det[F](Nat.0, a) = F.0 {
            F.0 * matrix2_det[F](Nat.0, b) = F.1
            F.0 * matrix2_det[F](Nat.0, b) = F.0
            F.0 = F.1
            F.0 != F.1
            false
        }
        matrix2_det[F](Nat.0, a) != F.0
    }
}

/// A matrix with nonzero determinant is invertible, with inverse
/// `(1/det(A)) * adj(A)`.
theorem matrix2_det_nonzero_implies_invertible[F: Field](a: Matrix[F, Nat.0.suc.suc, Nat.0.suc.suc]) {
    matrix2_det[F](Nat.0, a) != F.0 implies
        exists(b: Matrix[F, Nat.0.suc.suc, Nat.0.suc.suc]) {
            square_matrix_mul[F](Nat.0.suc.suc, a, b) = matrix_one[F](Nat.0.suc.suc) and
                square_matrix_mul[F](Nat.0.suc.suc, b, a) = matrix_one[F](Nat.0.suc.suc)
        }
} by {
    if matrix2_det[F](Nat.0, a) != F.0 {
        matrix2_inv_right[F](a)
        matrix2_inv_left[F](a)
        square_matrix_mul[F](Nat.0.suc.suc, a, matrix2_inv[F](Nat.0, a)) = matrix_one[F](Nat.0.suc.suc)
        square_matrix_mul[F](Nat.0.suc.suc, matrix2_inv[F](Nat.0, a), a) = matrix_one[F](Nat.0.suc.suc)
        exists(w: Matrix[F, Nat.0.suc.suc, Nat.0.suc.suc]) {
            w = matrix2_inv[F](Nat.0, a) and
                square_matrix_mul[F](Nat.0.suc.suc, a, w) = matrix_one[F](Nat.0.suc.suc) and
                square_matrix_mul[F](Nat.0.suc.suc, w, a) = matrix_one[F](Nat.0.suc.suc)
        }
        exists(w: Matrix[F, Nat.0.suc.suc, Nat.0.suc.suc]) {
            square_matrix_mul[F](Nat.0.suc.suc, a, w) = matrix_one[F](Nat.0.suc.suc) and
                square_matrix_mul[F](Nat.0.suc.suc, w, a) = matrix_one[F](Nat.0.suc.suc)
        }
    }
}

/// A two-by-two matrix is invertible exactly when its determinant is nonzero.
theorem matrix2_invertible_iff_det_nonzero[F: Field](a: Matrix[F, Nat.0.suc.suc, Nat.0.suc.suc]) {
    (exists(b: Matrix[F, Nat.0.suc.suc, Nat.0.suc.suc]) {
        square_matrix_mul[F](Nat.0.suc.suc, a, b) = matrix_one[F](Nat.0.suc.suc) and
            square_matrix_mul[F](Nat.0.suc.suc, b, a) = matrix_one[F](Nat.0.suc.suc)
    }) = (matrix2_det[F](Nat.0, a) != F.0)
} by {
    if exists(b: Matrix[F, Nat.0.suc.suc, Nat.0.suc.suc]) {
        square_matrix_mul[F](Nat.0.suc.suc, a, b) = matrix_one[F](Nat.0.suc.suc) and
            square_matrix_mul[F](Nat.0.suc.suc, b, a) = matrix_one[F](Nat.0.suc.suc)
    } {
        matrix2_invertible_implies_det_nonzero[F](a)
        matrix2_det[F](Nat.0, a) != F.0
    }
    if matrix2_det[F](Nat.0, a) != F.0 {
        matrix2_det_nonzero_implies_invertible[F](a)
        exists(b: Matrix[F, Nat.0.suc.suc, Nat.0.suc.suc]) {
            square_matrix_mul[F](Nat.0.suc.suc, a, b) = matrix_one[F](Nat.0.suc.suc) and
                square_matrix_mul[F](Nat.0.suc.suc, b, a) = matrix_one[F](Nat.0.suc.suc)
        }
    }
}

// ---------------------------------------------------------------------------
// Cramer's rule
// ---------------------------------------------------------------------------

/// An entry of a matrix applied to a vector is the two-term sum of products.
theorem matrix2_apply_entry[F: Semiring](a: Matrix[F, Nat.0.suc.suc, Nat.0.suc.suc], v: Fin[Nat.0.suc.suc] -> F, i: Fin[Nat.0.suc.suc]) {
    matrix_apply[F](Nat.0.suc.suc, a, v, i) =
        matrix_entry[F](Nat.0.suc.suc, Nat.0.suc.suc, a, i, fin_zero(Nat.0.suc)) * v(fin_zero(Nat.0.suc)) +
        matrix_entry[F](Nat.0.suc.suc, Nat.0.suc.suc, a, i, fin_succ(Nat.0.suc, fin_zero(Nat.0))) * v(fin_succ(Nat.0.suc, fin_zero(Nat.0)))
} by {
    matrix_apply_eq[F](Nat.0.suc.suc, a, v, i)
    matrix_apply[F](Nat.0.suc.suc, a, v, i) = fin_sum[F](Nat.0.suc.suc, matrix_apply_term[F](Nat.0.suc.suc, a, v, i))
    fin_sum_two[F](matrix_apply_term[F](Nat.0.suc.suc, a, v, i))
    fin_sum[F](Nat.0.suc.suc, matrix_apply_term[F](Nat.0.suc.suc, a, v, i)) =
        matrix_apply_term[F](Nat.0.suc.suc, a, v, i, fin_zero(Nat.0.suc)) +
        matrix_apply_term[F](Nat.0.suc.suc, a, v, i, fin_succ(Nat.0.suc, fin_zero(Nat.0)))
    matrix_apply_term[F](Nat.0.suc.suc, a, v, i, fin_zero(Nat.0.suc)) =
        matrix_entry[F](Nat.0.suc.suc, Nat.0.suc.suc, a, i, fin_zero(Nat.0.suc)) * v(fin_zero(Nat.0.suc))
    matrix_apply_term[F](Nat.0.suc.suc, a, v, i, fin_succ(Nat.0.suc, fin_zero(Nat.0))) =
        matrix_entry[F](Nat.0.suc.suc, Nat.0.suc.suc, a, i, fin_succ(Nat.0.suc, fin_zero(Nat.0))) * v(fin_succ(Nat.0.suc, fin_zero(Nat.0)))
    matrix_apply[F](Nat.0.suc.suc, a, v, i) =
        matrix_entry[F](Nat.0.suc.suc, Nat.0.suc.suc, a, i, fin_zero(Nat.0.suc)) * v(fin_zero(Nat.0.suc)) +
        matrix_entry[F](Nat.0.suc.suc, Nat.0.suc.suc, a, i, fin_succ(Nat.0.suc, fin_zero(Nat.0))) * v(fin_succ(Nat.0.suc, fin_zero(Nat.0)))
}

/// Applying the identity matrix to a vector changes nothing.
theorem matrix2_apply_one[F: Semiring](v: Fin[Nat.0.suc.suc] -> F, i: Fin[Nat.0.suc.suc]) {
    matrix_apply[F](Nat.0.suc.suc, matrix_one[F](Nat.0.suc.suc), v, i) = v(i)
} by {
    matrix2_apply_entry[F](matrix_one[F](Nat.0.suc.suc), v, i)
    matrix_apply[F](Nat.0.suc.suc, matrix_one[F](Nat.0.suc.suc), v, i) =
        matrix_entry[F](Nat.0.suc.suc, Nat.0.suc.suc, matrix_one[F](Nat.0.suc.suc), i, fin_zero(Nat.0.suc)) * v(fin_zero(Nat.0.suc)) +
        matrix_entry[F](Nat.0.suc.suc, Nat.0.suc.suc, matrix_one[F](Nat.0.suc.suc), i, fin_succ(Nat.0.suc, fin_zero(Nat.0))) * v(fin_succ(Nat.0.suc, fin_zero(Nat.0)))
    if i.value = Nat.0 {
        fin_two_zero_value_two
        i.value = fin_zero(Nat.0.suc).value
        ext(Nat.0.suc.suc, i, fin_zero(Nat.0.suc))
        i = fin_zero(Nat.0.suc)
        matrix_entry_one_diag[F](Nat.0.suc.suc, fin_zero(Nat.0.suc))
        matrix_entry_one_off_diag[F](Nat.0.suc.suc, fin_zero(Nat.0.suc), fin_succ(Nat.0.suc, fin_zero(Nat.0)))
        matrix_entry[F](Nat.0.suc.suc, Nat.0.suc.suc, matrix_one[F](Nat.0.suc.suc), i, fin_zero(Nat.0.suc)) = F.1
        matrix_entry[F](Nat.0.suc.suc, Nat.0.suc.suc, matrix_one[F](Nat.0.suc.suc), i, fin_succ(Nat.0.suc, fin_zero(Nat.0))) = F.0
        matrix_apply[F](Nat.0.suc.suc, matrix_one[F](Nat.0.suc.suc), v, i) = F.1 * v(fin_zero(Nat.0.suc)) + F.0 * v(fin_succ(Nat.0.suc, fin_zero(Nat.0)))
        F.1 * v(fin_zero(Nat.0.suc)) = v(fin_zero(Nat.0.suc))
        F.0 * v(fin_succ(Nat.0.suc, fin_zero(Nat.0))) = F.0
        v(fin_zero(Nat.0.suc)) + F.0 = v(fin_zero(Nat.0.suc))
        matrix_apply[F](Nat.0.suc.suc, matrix_one[F](Nat.0.suc.suc), v, i) = v(fin_zero(Nat.0.suc))
        matrix_apply[F](Nat.0.suc.suc, matrix_one[F](Nat.0.suc.suc), v, i) = v(i)
    } else {
        fin_two_one_value_two
        i.value = fin_succ(Nat.0.suc, fin_zero(Nat.0)).value
        ext(Nat.0.suc.suc, i, fin_succ(Nat.0.suc, fin_zero(Nat.0)))
        i = fin_succ(Nat.0.suc, fin_zero(Nat.0))
        matrix_entry_one_off_diag[F](Nat.0.suc.suc, fin_succ(Nat.0.suc, fin_zero(Nat.0)), fin_zero(Nat.0.suc))
        matrix_entry_one_diag[F](Nat.0.suc.suc, fin_succ(Nat.0.suc, fin_zero(Nat.0)))
        matrix_entry[F](Nat.0.suc.suc, Nat.0.suc.suc, matrix_one[F](Nat.0.suc.suc), i, fin_zero(Nat.0.suc)) = F.0
        matrix_entry[F](Nat.0.suc.suc, Nat.0.suc.suc, matrix_one[F](Nat.0.suc.suc), i, fin_succ(Nat.0.suc, fin_zero(Nat.0))) = F.1
        matrix_apply[F](Nat.0.suc.suc, matrix_one[F](Nat.0.suc.suc), v, i) = F.0 * v(fin_zero(Nat.0.suc)) + F.1 * v(fin_succ(Nat.0.suc, fin_zero(Nat.0)))
        F.0 * v(fin_zero(Nat.0.suc)) = F.0
        F.1 * v(fin_succ(Nat.0.suc, fin_zero(Nat.0))) = v(fin_succ(Nat.0.suc, fin_zero(Nat.0)))
        F.0 + v(fin_succ(Nat.0.suc, fin_zero(Nat.0))) = v(fin_succ(Nat.0.suc, fin_zero(Nat.0)))
        matrix_apply[F](Nat.0.suc.suc, matrix_one[F](Nat.0.suc.suc), v, i) = v(fin_succ(Nat.0.suc, fin_zero(Nat.0)))
        matrix_apply[F](Nat.0.suc.suc, matrix_one[F](Nat.0.suc.suc), v, i) = v(i)
    }
}

/// The entry of a replaced column at the replaced position is the vector
/// entry.
theorem matrix2_replace_col_entry_self[F: Ring](
    a: Matrix[F, Nat.0.suc.suc, Nat.0.suc.suc], j: Fin[Nat.0.suc.suc], b: Fin[Nat.0.suc.suc] -> F, i: Fin[Nat.0.suc.suc]
) {
    matrix_entry[F](Nat.0.suc.suc, Nat.0.suc.suc, matrix2_replace_col[F](Nat.0, a, j, b), i, j) = b(i)
} by {
    matrix_entry[F](Nat.0.suc.suc, Nat.0.suc.suc, matrix2_replace_col[F](Nat.0, a, j, b), i, j) =
        matrix2_replace_col_entry[F](Nat.0, a, j, b, i, j)
    matrix2_replace_col_entry[F](Nat.0, a, j, b, i, j) = b(i)
    matrix_entry[F](Nat.0.suc.suc, Nat.0.suc.suc, matrix2_replace_col[F](Nat.0, a, j, b), i, j) = b(i)
}

/// The entry of a replaced column away from the replaced position is the
/// original entry.
theorem matrix2_replace_col_entry_other[F: Ring](
    a: Matrix[F, Nat.0.suc.suc, Nat.0.suc.suc], j: Fin[Nat.0.suc.suc], b: Fin[Nat.0.suc.suc] -> F, i: Fin[Nat.0.suc.suc], k: Fin[Nat.0.suc.suc]
) {
    k != j implies matrix_entry[F](Nat.0.suc.suc, Nat.0.suc.suc, matrix2_replace_col[F](Nat.0, a, j, b), i, k) = matrix_entry[F](Nat.0.suc.suc, Nat.0.suc.suc, a, i, k)
} by {
    if k != j {
        matrix_entry[F](Nat.0.suc.suc, Nat.0.suc.suc, matrix2_replace_col[F](Nat.0, a, j, b), i, k) =
            matrix2_replace_col_entry[F](Nat.0, a, j, b, i, k)
        matrix2_replace_col_entry[F](Nat.0, a, j, b, i, k) = matrix_entry[F](Nat.0.suc.suc, Nat.0.suc.suc, a, i, k)
        matrix_entry[F](Nat.0.suc.suc, Nat.0.suc.suc, matrix2_replace_col[F](Nat.0, a, j, b), i, k) =
            matrix_entry[F](Nat.0.suc.suc, Nat.0.suc.suc, a, i, k)
    }
}

/// Cramer's rule: the `i`th entry of the solution `inv(A) * b` is the
/// determinant of `A` with column `i` replaced by `b`, divided by the
/// determinant of `A`.
theorem matrix2_cramer[F: Field](a: Matrix[F, Nat.0.suc.suc, Nat.0.suc.suc], b: Fin[Nat.0.suc.suc] -> F, i: Fin[Nat.0.suc.suc]) {
    matrix2_det[F](Nat.0, a) != F.0 implies
        matrix_apply[F](Nat.0.suc.suc, matrix2_inv[F](Nat.0, a), b, i) =
            matrix2_det[F](Nat.0, matrix2_replace_col[F](Nat.0, a, i, b)) * matrix2_det[F](Nat.0, a).inverse
} by {
    if matrix2_det[F](Nat.0, a) != F.0 {
        if i.value = Nat.0 {
            fin_two_zero_value_two
            i.value = fin_zero(Nat.0.suc).value
            ext(Nat.0.suc.suc, i, fin_zero(Nat.0.suc))
            i = fin_zero(Nat.0.suc)
            matrix2_apply_entry[F](matrix2_inv[F](Nat.0, a), b, fin_zero(Nat.0.suc))
            matrix_apply[F](Nat.0.suc.suc, matrix2_inv[F](Nat.0, a), b, fin_zero(Nat.0.suc)) =
                matrix_entry[F](Nat.0.suc.suc, Nat.0.suc.suc, matrix2_inv[F](Nat.0, a), fin_zero(Nat.0.suc), fin_zero(Nat.0.suc)) * b(fin_zero(Nat.0.suc)) +
                matrix_entry[F](Nat.0.suc.suc, Nat.0.suc.suc, matrix2_inv[F](Nat.0, a), fin_zero(Nat.0.suc), fin_succ(Nat.0.suc, fin_zero(Nat.0))) * b(fin_succ(Nat.0.suc, fin_zero(Nat.0)))
            matrix2_inv_matrix_entry[F](a, fin_zero(Nat.0.suc), fin_zero(Nat.0.suc))
            matrix2_inv_matrix_entry[F](a, fin_zero(Nat.0.suc), fin_succ(Nat.0.suc, fin_zero(Nat.0)))
            matrix2_inv_value_00[F](a)
            matrix2_inv_value_01[F](a)
            matrix_apply[F](Nat.0.suc.suc, matrix2_inv[F](Nat.0, a), b, fin_zero(Nat.0.suc)) =
                (matrix2_det[F](Nat.0, a).inverse * matrix2_entry_11[F](Nat.0, a)) * b(fin_zero(Nat.0.suc)) +
                (matrix2_det[F](Nat.0, a).inverse * (-matrix2_entry_01[F](Nat.0, a))) * b(fin_succ(Nat.0.suc, fin_zero(Nat.0)))
            (matrix2_det[F](Nat.0, a).inverse * matrix2_entry_11[F](Nat.0, a)) * b(fin_zero(Nat.0.suc)) =
                matrix2_det[F](Nat.0, a).inverse * (matrix2_entry_11[F](Nat.0, a) * b(fin_zero(Nat.0.suc)))
            (matrix2_det[F](Nat.0, a).inverse * (-matrix2_entry_01[F](Nat.0, a))) * b(fin_succ(Nat.0.suc, fin_zero(Nat.0))) =
                matrix2_det[F](Nat.0, a).inverse * ((-matrix2_entry_01[F](Nat.0, a)) * b(fin_succ(Nat.0.suc, fin_zero(Nat.0))))
            (-matrix2_entry_01[F](Nat.0, a)) * b(fin_succ(Nat.0.suc, fin_zero(Nat.0))) =
                -(matrix2_entry_01[F](Nat.0, a) * b(fin_succ(Nat.0.suc, fin_zero(Nat.0))))
            matrix2_entry_11[F](Nat.0, a) * b(fin_zero(Nat.0.suc)) =
                b(fin_zero(Nat.0.suc)) * matrix2_entry_11[F](Nat.0, a)
            matrix_apply[F](Nat.0.suc.suc, matrix2_inv[F](Nat.0, a), b, fin_zero(Nat.0.suc)) =
                matrix2_det[F](Nat.0, a).inverse * (matrix2_entry_11[F](Nat.0, a) * b(fin_zero(Nat.0.suc))) +
                matrix2_det[F](Nat.0, a).inverse * (-(matrix2_entry_01[F](Nat.0, a) * b(fin_succ(Nat.0.suc, fin_zero(Nat.0)))))
            matrix2_det[F](Nat.0, a).inverse * (matrix2_entry_11[F](Nat.0, a) * b(fin_zero(Nat.0.suc))) +
                matrix2_det[F](Nat.0, a).inverse * (-(matrix2_entry_01[F](Nat.0, a) * b(fin_succ(Nat.0.suc, fin_zero(Nat.0))))) =
                matrix2_det[F](Nat.0, a).inverse *
                    (matrix2_entry_11[F](Nat.0, a) * b(fin_zero(Nat.0.suc)) -
                        matrix2_entry_01[F](Nat.0, a) * b(fin_succ(Nat.0.suc, fin_zero(Nat.0))))
            matrix_apply[F](Nat.0.suc.suc, matrix2_inv[F](Nat.0, a), b, fin_zero(Nat.0.suc)) =
                matrix2_det[F](Nat.0, a).inverse *
                    (b(fin_zero(Nat.0.suc)) * matrix2_entry_11[F](Nat.0, a) -
                        matrix2_entry_01[F](Nat.0, a) * b(fin_succ(Nat.0.suc, fin_zero(Nat.0))))
            matrix2_det_unfold[F](matrix2_replace_col[F](Nat.0, a, fin_zero(Nat.0.suc), b))
            matrix2_replace_col_entry_self[F](a, fin_zero(Nat.0.suc), b, fin_zero(Nat.0.suc))
            matrix2_replace_col_entry_other[F](a, fin_zero(Nat.0.suc), b, fin_succ(Nat.0.suc, fin_zero(Nat.0)), fin_succ(Nat.0.suc, fin_zero(Nat.0)))
            matrix2_replace_col_entry_other[F](a, fin_zero(Nat.0.suc), b, fin_zero(Nat.0.suc), fin_succ(Nat.0.suc, fin_zero(Nat.0)))
            matrix2_replace_col_entry_self[F](a, fin_zero(Nat.0.suc), b, fin_succ(Nat.0.suc, fin_zero(Nat.0)))
            fin_two_zero_ne_one
            matrix2_entry_00[F](Nat.0, matrix2_replace_col[F](Nat.0, a, fin_zero(Nat.0.suc), b)) =
                b(fin_zero(Nat.0.suc))
            matrix2_entry_11[F](Nat.0, matrix2_replace_col[F](Nat.0, a, fin_zero(Nat.0.suc), b)) =
                matrix2_entry_11[F](Nat.0, a)
            matrix2_entry_01[F](Nat.0, matrix2_replace_col[F](Nat.0, a, fin_zero(Nat.0.suc), b)) =
                matrix2_entry_01[F](Nat.0, a)
            matrix2_entry_10[F](Nat.0, matrix2_replace_col[F](Nat.0, a, fin_zero(Nat.0.suc), b)) =
                b(fin_succ(Nat.0.suc, fin_zero(Nat.0)))
            matrix2_det[F](Nat.0, matrix2_replace_col[F](Nat.0, a, fin_zero(Nat.0.suc), b)) =
                b(fin_zero(Nat.0.suc)) * matrix2_entry_11[F](Nat.0, a) -
                    matrix2_entry_01[F](Nat.0, a) * b(fin_succ(Nat.0.suc, fin_zero(Nat.0)))
            matrix_apply[F](Nat.0.suc.suc, matrix2_inv[F](Nat.0, a), b, fin_zero(Nat.0.suc)) =
                matrix2_det[F](Nat.0, a).inverse * matrix2_det[F](Nat.0, matrix2_replace_col[F](Nat.0, a, fin_zero(Nat.0.suc), b))
            matrix2_det[F](Nat.0, a).inverse * matrix2_det[F](Nat.0, matrix2_replace_col[F](Nat.0, a, fin_zero(Nat.0.suc), b)) =
                matrix2_det[F](Nat.0, matrix2_replace_col[F](Nat.0, a, fin_zero(Nat.0.suc), b)) * matrix2_det[F](Nat.0, a).inverse
            matrix_apply[F](Nat.0.suc.suc, matrix2_inv[F](Nat.0, a), b, i) =
                matrix2_det[F](Nat.0, matrix2_replace_col[F](Nat.0, a, i, b)) * matrix2_det[F](Nat.0, a).inverse
        } else {
            fin_two_one_value_two
            i.value = fin_succ(Nat.0.suc, fin_zero(Nat.0)).value
            ext(Nat.0.suc.suc, i, fin_succ(Nat.0.suc, fin_zero(Nat.0)))
            i = fin_succ(Nat.0.suc, fin_zero(Nat.0))
            matrix2_apply_entry[F](matrix2_inv[F](Nat.0, a), b, fin_succ(Nat.0.suc, fin_zero(Nat.0)))
            matrix_apply[F](Nat.0.suc.suc, matrix2_inv[F](Nat.0, a), b, fin_succ(Nat.0.suc, fin_zero(Nat.0))) =
                matrix_entry[F](Nat.0.suc.suc, Nat.0.suc.suc, matrix2_inv[F](Nat.0, a), fin_succ(Nat.0.suc, fin_zero(Nat.0)), fin_zero(Nat.0.suc)) * b(fin_zero(Nat.0.suc)) +
                matrix_entry[F](Nat.0.suc.suc, Nat.0.suc.suc, matrix2_inv[F](Nat.0, a), fin_succ(Nat.0.suc, fin_zero(Nat.0)), fin_succ(Nat.0.suc, fin_zero(Nat.0))) * b(fin_succ(Nat.0.suc, fin_zero(Nat.0)))
            matrix2_inv_matrix_entry[F](a, fin_succ(Nat.0.suc, fin_zero(Nat.0)), fin_zero(Nat.0.suc))
            matrix2_inv_matrix_entry[F](a, fin_succ(Nat.0.suc, fin_zero(Nat.0)), fin_succ(Nat.0.suc, fin_zero(Nat.0)))
            matrix2_inv_value_10[F](a)
            matrix2_inv_value_11[F](a)
            matrix_apply[F](Nat.0.suc.suc, matrix2_inv[F](Nat.0, a), b, fin_succ(Nat.0.suc, fin_zero(Nat.0))) =
                (matrix2_det[F](Nat.0, a).inverse * (-matrix2_entry_10[F](Nat.0, a))) * b(fin_zero(Nat.0.suc)) +
                (matrix2_det[F](Nat.0, a).inverse * matrix2_entry_00[F](Nat.0, a)) * b(fin_succ(Nat.0.suc, fin_zero(Nat.0)))
            (matrix2_det[F](Nat.0, a).inverse * (-matrix2_entry_10[F](Nat.0, a))) * b(fin_zero(Nat.0.suc)) =
                matrix2_det[F](Nat.0, a).inverse * ((-matrix2_entry_10[F](Nat.0, a)) * b(fin_zero(Nat.0.suc)))
            (matrix2_det[F](Nat.0, a).inverse * matrix2_entry_00[F](Nat.0, a)) * b(fin_succ(Nat.0.suc, fin_zero(Nat.0))) =
                matrix2_det[F](Nat.0, a).inverse * (matrix2_entry_00[F](Nat.0, a) * b(fin_succ(Nat.0.suc, fin_zero(Nat.0))))
            (-matrix2_entry_10[F](Nat.0, a)) * b(fin_zero(Nat.0.suc)) =
                -(matrix2_entry_10[F](Nat.0, a) * b(fin_zero(Nat.0.suc)))
            matrix_apply[F](Nat.0.suc.suc, matrix2_inv[F](Nat.0, a), b, fin_succ(Nat.0.suc, fin_zero(Nat.0))) =
                matrix2_det[F](Nat.0, a).inverse * (-(matrix2_entry_10[F](Nat.0, a) * b(fin_zero(Nat.0.suc)))) +
                matrix2_det[F](Nat.0, a).inverse * (matrix2_entry_00[F](Nat.0, a) * b(fin_succ(Nat.0.suc, fin_zero(Nat.0))))
            matrix2_det[F](Nat.0, a).inverse * (-(matrix2_entry_10[F](Nat.0, a) * b(fin_zero(Nat.0.suc)))) +
                matrix2_det[F](Nat.0, a).inverse * (matrix2_entry_00[F](Nat.0, a) * b(fin_succ(Nat.0.suc, fin_zero(Nat.0)))) =
                matrix2_det[F](Nat.0, a).inverse *
                    (-(matrix2_entry_10[F](Nat.0, a) * b(fin_zero(Nat.0.suc))) +
                        matrix2_entry_00[F](Nat.0, a) * b(fin_succ(Nat.0.suc, fin_zero(Nat.0))))
            matrix2_det[F](Nat.0, a).inverse *
                (-(matrix2_entry_10[F](Nat.0, a) * b(fin_zero(Nat.0.suc))) +
                    matrix2_entry_00[F](Nat.0, a) * b(fin_succ(Nat.0.suc, fin_zero(Nat.0)))) =
                matrix2_det[F](Nat.0, a).inverse *
                    (matrix2_entry_00[F](Nat.0, a) * b(fin_succ(Nat.0.suc, fin_zero(Nat.0))) -
                        matrix2_entry_10[F](Nat.0, a) * b(fin_zero(Nat.0.suc)))
            matrix_apply[F](Nat.0.suc.suc, matrix2_inv[F](Nat.0, a), b, fin_succ(Nat.0.suc, fin_zero(Nat.0))) =
                matrix2_det[F](Nat.0, a).inverse *
                    (matrix2_entry_00[F](Nat.0, a) * b(fin_succ(Nat.0.suc, fin_zero(Nat.0))) -
                        matrix2_entry_10[F](Nat.0, a) * b(fin_zero(Nat.0.suc)))
            matrix2_det_unfold[F](matrix2_replace_col[F](Nat.0, a, fin_succ(Nat.0.suc, fin_zero(Nat.0)), b))
            matrix2_replace_col_entry_other[F](a, fin_succ(Nat.0.suc, fin_zero(Nat.0)), b, fin_zero(Nat.0.suc), fin_zero(Nat.0.suc))
            matrix2_replace_col_entry_self[F](a, fin_succ(Nat.0.suc, fin_zero(Nat.0)), b, fin_succ(Nat.0.suc, fin_zero(Nat.0)))
            matrix2_replace_col_entry_self[F](a, fin_succ(Nat.0.suc, fin_zero(Nat.0)), b, fin_zero(Nat.0.suc))
            matrix2_replace_col_entry_other[F](a, fin_succ(Nat.0.suc, fin_zero(Nat.0)), b, fin_succ(Nat.0.suc, fin_zero(Nat.0)), fin_zero(Nat.0.suc))
            fin_two_one_ne_zero
            matrix2_entry_00[F](Nat.0, matrix2_replace_col[F](Nat.0, a, fin_succ(Nat.0.suc, fin_zero(Nat.0)), b)) =
                matrix2_entry_00[F](Nat.0, a)
            matrix2_entry_11[F](Nat.0, matrix2_replace_col[F](Nat.0, a, fin_succ(Nat.0.suc, fin_zero(Nat.0)), b)) =
                b(fin_succ(Nat.0.suc, fin_zero(Nat.0)))
            matrix2_entry_01[F](Nat.0, matrix2_replace_col[F](Nat.0, a, fin_succ(Nat.0.suc, fin_zero(Nat.0)), b)) =
                b(fin_zero(Nat.0.suc))
            matrix2_entry_10[F](Nat.0, matrix2_replace_col[F](Nat.0, a, fin_succ(Nat.0.suc, fin_zero(Nat.0)), b)) =
                matrix2_entry_10[F](Nat.0, a)
            matrix2_det[F](Nat.0, matrix2_replace_col[F](Nat.0, a, fin_succ(Nat.0.suc, fin_zero(Nat.0)), b)) =
                matrix2_entry_00[F](Nat.0, a) * b(fin_succ(Nat.0.suc, fin_zero(Nat.0))) -
                    matrix2_entry_10[F](Nat.0, a) * b(fin_zero(Nat.0.suc))
            matrix_apply[F](Nat.0.suc.suc, matrix2_inv[F](Nat.0, a), b, fin_succ(Nat.0.suc, fin_zero(Nat.0))) =
                matrix2_det[F](Nat.0, a).inverse * matrix2_det[F](Nat.0, matrix2_replace_col[F](Nat.0, a, fin_succ(Nat.0.suc, fin_zero(Nat.0)), b))
            matrix2_det[F](Nat.0, a).inverse * matrix2_det[F](Nat.0, matrix2_replace_col[F](Nat.0, a, fin_succ(Nat.0.suc, fin_zero(Nat.0)), b)) =
                matrix2_det[F](Nat.0, matrix2_replace_col[F](Nat.0, a, fin_succ(Nat.0.suc, fin_zero(Nat.0)), b)) * matrix2_det[F](Nat.0, a).inverse
            matrix_apply[F](Nat.0.suc.suc, matrix2_inv[F](Nat.0, a), b, i) =
                matrix2_det[F](Nat.0, matrix2_replace_col[F](Nat.0, a, i, b)) * matrix2_det[F](Nat.0, a).inverse
        }
    }
}

/// The Cramer solution `inv(A) * b` solves `A x = b`.
theorem matrix2_cramer_solves[F: Field](a: Matrix[F, Nat.0.suc.suc, Nat.0.suc.suc], b: Fin[Nat.0.suc.suc] -> F, i: Fin[Nat.0.suc.suc]) {
    matrix2_det[F](Nat.0, a) != F.0 implies
        matrix_apply[F](Nat.0.suc.suc, a, matrix_apply_vec[F](Nat.0.suc.suc, matrix2_inv[F](Nat.0, a), b), i) = b(i)
} by {
    if matrix2_det[F](Nat.0, a) != F.0 {
        matrix_apply_mul_two[F](a, matrix2_inv[F](Nat.0, a), b, i)
        matrix_apply[F](Nat.0.suc.suc, square_matrix_mul[F](Nat.0.suc.suc, a, matrix2_inv[F](Nat.0, a)), b, i) =
            matrix_apply[F](Nat.0.suc.suc, a, matrix_apply_vec[F](Nat.0.suc.suc, matrix2_inv[F](Nat.0, a), b), i)
        matrix2_inv_right[F](a)
        square_matrix_mul[F](Nat.0.suc.suc, a, matrix2_inv[F](Nat.0, a)) = matrix_one[F](Nat.0.suc.suc)
        matrix_apply[F](Nat.0.suc.suc, square_matrix_mul[F](Nat.0.suc.suc, a, matrix2_inv[F](Nat.0, a)), b, i) =
            matrix_apply[F](Nat.0.suc.suc, matrix_one[F](Nat.0.suc.suc), b, i)
        matrix2_apply_one[F](b, i)
        matrix_apply[F](Nat.0.suc.suc, matrix_one[F](Nat.0.suc.suc), b, i) = b(i)
        matrix_apply[F](Nat.0.suc.suc, a, matrix_apply_vec[F](Nat.0.suc.suc, matrix2_inv[F](Nat.0, a), b), i) = b(i)
    }
}

/// The solution of `A x = b` is unique: any solution agrees with
/// `inv(A) * b`.
theorem matrix2_solution_unique[F: Field](a: Matrix[F, Nat.0.suc.suc, Nat.0.suc.suc], b: Fin[Nat.0.suc.suc] -> F, y: Fin[Nat.0.suc.suc] -> F, i: Fin[Nat.0.suc.suc]) {
    matrix2_det[F](Nat.0, a) != F.0 implies
        ((forall(k: Fin[Nat.0.suc.suc]) { matrix_apply[F](Nat.0.suc.suc, a, y, k) = b(k) }) implies
            y(i) = matrix_apply[F](Nat.0.suc.suc, matrix2_inv[F](Nat.0, a), b, i))
} by {
    if matrix2_det[F](Nat.0, a) != F.0 {
        if forall(k: Fin[Nat.0.suc.suc]) { matrix_apply[F](Nat.0.suc.suc, a, y, k) = b(k) } {
            matrix_apply_mul_two[F](matrix2_inv[F](Nat.0, a), a, y, i)
            matrix_apply[F](Nat.0.suc.suc, square_matrix_mul[F](Nat.0.suc.suc, matrix2_inv[F](Nat.0, a), a), y, i) =
                matrix_apply[F](Nat.0.suc.suc, matrix2_inv[F](Nat.0, a), matrix_apply_vec[F](Nat.0.suc.suc, a, y), i)
            matrix2_inv_left[F](a)
            square_matrix_mul[F](Nat.0.suc.suc, matrix2_inv[F](Nat.0, a), a) = matrix_one[F](Nat.0.suc.suc)
            matrix_apply[F](Nat.0.suc.suc, square_matrix_mul[F](Nat.0.suc.suc, matrix2_inv[F](Nat.0, a), a), y, i) =
                matrix_apply[F](Nat.0.suc.suc, matrix_one[F](Nat.0.suc.suc), y, i)
            matrix2_apply_one[F](y, i)
            matrix_apply[F](Nat.0.suc.suc, matrix_one[F](Nat.0.suc.suc), y, i) = y(i)
            forall(k: Fin[Nat.0.suc.suc]) {
                matrix_apply_vec[F](Nat.0.suc.suc, a, y, k) = matrix_apply[F](Nat.0.suc.suc, a, y, k)
                matrix_apply[F](Nat.0.suc.suc, a, y, k) = b(k)
                matrix_apply_vec[F](Nat.0.suc.suc, a, y, k) = b(k)
            }
            matrix_apply_pointwise_eq[F](Nat.0.suc.suc, matrix2_inv[F](Nat.0, a), matrix_apply_vec[F](Nat.0.suc.suc, a, y), b, i)
            matrix_apply[F](Nat.0.suc.suc, matrix2_inv[F](Nat.0, a), matrix_apply_vec[F](Nat.0.suc.suc, a, y), i) =
                matrix_apply[F](Nat.0.suc.suc, matrix2_inv[F](Nat.0, a), b, i)
            matrix_apply[F](Nat.0.suc.suc, matrix2_inv[F](Nat.0, a), matrix_apply_vec[F](Nat.0.suc.suc, a, y), i) = y(i)
            y(i) = matrix_apply[F](Nat.0.suc.suc, matrix2_inv[F](Nat.0, a), b, i)
        }
    }
}
