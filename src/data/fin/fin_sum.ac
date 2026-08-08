from nat import Nat
from data.fin.fin import Fin, new_self, fin_new_some_imp_lt
from algebra.zero import Zero
from algebra.add import Add
from algebra.add_comm_monoid import AddCommMonoid
from algebra.add_semigroup import add_fn
from data.basic.functions import function_extensionality
from list import partial, partial_add, partial_pointwise_eq, partial_split_last, partial_zero

/// The optional element of `Fin[n]` represented by a natural number.
define fin_of_nat_option(n: Nat, k: Nat) -> Option[Fin[n]] {
    Fin[n].new(k)
}

/// The value of a function on `Fin[n]` at the natural index `k`, with zero outside the range.
define fin_value_or_zero[A: Zero](n: Nat, f: Fin[n] -> A, k: Nat) -> A {
    match fin_of_nat_option(n, k) {
        Option.none {
            A.0
        }
        Option.some(i) {
            f(i)
        }
    }
}

/// Evaluating the zero extension at the natural value of a finite index recovers the function value.
theorem fin_value_or_zero_value[A: Zero](n: Nat, f: Fin[n] -> A, i: Fin[n]) {
    fin_value_or_zero[A](n, f, i.value) = f(i)
} by {
    new_self(n, i)
}

/// Pointwise addition of two functions on `Fin[n]`.
define fin_pointwise_add[A: Add](n: Nat, f: Fin[n] -> A, g: Fin[n] -> A, i: Fin[n]) -> A {
    f(i) + g(i)
}

/// The zero function on `Fin[n]`.
define fin_zero_function[A: Zero](n: Nat, i: Fin[n]) -> A {
    A.0
}

/// The zero function on natural numbers.
define nat_zero_function[A: Zero](k: Nat) -> A {
    A.0
}


/// The finite sum of a function on `Fin[n]`.
define fin_sum[A: AddCommMonoid](n: Nat, f: Fin[n] -> A) -> A {
    partial(fin_value_or_zero[A](n, f), n)
}

/// The finite sum is the partial sum of the zero-extended natural-indexed function.
theorem fin_sum_apply[A: AddCommMonoid](n: Nat, f: Fin[n] -> A) {
    fin_sum[A](n, f) = partial(fin_value_or_zero[A](n, f), n)
}

/// Zero extension commutes with pointwise addition on `Fin[n]`.
theorem fin_value_or_zero_add[A: AddCommMonoid](n: Nat, f: Fin[n] -> A, g: Fin[n] -> A, k: Nat) {
    fin_value_or_zero[A](n, fin_pointwise_add[A](n, f, g), k) =
        fin_value_or_zero[A](n, f, k) + fin_value_or_zero[A](n, g, k)
} by {
    match fin_of_nat_option(n, k) {
        Option.none {
            fin_value_or_zero[A](n, fin_pointwise_add[A](n, f, g), k) = A.0
            fin_value_or_zero[A](n, f, k) = A.0
            fin_value_or_zero[A](n, g, k) = A.0
            fin_value_or_zero[A](n, fin_pointwise_add[A](n, f, g), k) =
                fin_value_or_zero[A](n, f, k) + fin_value_or_zero[A](n, g, k)
        }
        Option.some(i) {
            fin_value_or_zero[A](n, fin_pointwise_add[A](n, f, g), k) = fin_pointwise_add[A](n, f, g, i)
            fin_value_or_zero[A](n, f, k) = f(i)
            fin_value_or_zero[A](n, g, k) = g(i)
            fin_value_or_zero[A](n, fin_pointwise_add[A](n, f, g), k) =
                fin_value_or_zero[A](n, f, k) + fin_value_or_zero[A](n, g, k)
        }
    }
}

/// Finite sums distribute over pointwise addition.
theorem fin_sum_add[A: AddCommMonoid](n: Nat, f: Fin[n] -> A, g: Fin[n] -> A) {
    fin_sum[A](n, fin_pointwise_add[A](n, f, g)) = fin_sum[A](n, f) + fin_sum[A](n, g)
} by {
    let h = fin_value_or_zero[A](n, fin_pointwise_add[A](n, f, g))
    let fz = fin_value_or_zero[A](n, f)
    let gz = fin_value_or_zero[A](n, g)
    forall(k: Nat) {
        if k < n {
            fin_value_or_zero_add[A](n, f, g, k)
            h(k) = add_fn(fz, gz, k)
        }
    }
    partial_pointwise_eq[A](h, add_fn(fz, gz), n)
    partial(h, n) = partial(add_fn(fz, gz), n)
    partial_add[A](fz, gz, n)
}

/// Pointwise equal functions on `Fin[n]` have equal finite sums.
theorem fin_sum_pointwise_eq[A: AddCommMonoid](n: Nat, f: Fin[n] -> A, g: Fin[n] -> A) {
    (forall(i: Fin[n]) { f(i) = g(i) }) implies fin_sum[A](n, f) = fin_sum[A](n, g)
} by {
    if forall(i: Fin[n]) { f(i) = g(i) } {
        function_extensionality(f, g)
    }
}

/// If the bound is zero, the finite sum is zero.
theorem fin_sum_zero_of_bound_zero[A: AddCommMonoid](n: Nat, f: Fin[n] -> A) {
    n = Nat.0 implies fin_sum[A](n, f) = A.0
} by {
    if n = Nat.0 {
        partial_zero[A](fin_value_or_zero[A](n, f))
        fin_sum[A](n, f) = A.0
    }
}

/// The partial sum of the zero natural-number function is zero.
theorem partial_nat_zero_function[A: AddCommMonoid](n: Nat) {
    partial(nat_zero_function[A], n) = A.0
} by {
    define p(k: Nat) -> Bool {
        partial(nat_zero_function[A], k) = A.0
    }
    partial_zero[A](nat_zero_function[A])
    p(Nat.0)
    forall(k: Nat) {
        if p(k) {
            partial_split_last[A](nat_zero_function[A], k)
            partial(nat_zero_function[A], k) = A.0
            p(k.suc)
        }
    }
    p(n)
}

/// Zero extension of the zero `Fin[n]` function is the zero natural-number function.
theorem fin_value_or_zero_zero_function[A: AddCommMonoid](n: Nat, k: Nat) {
    fin_value_or_zero[A](n, fin_zero_function[A](n), k) = nat_zero_function[A](k)
} by {
    match fin_of_nat_option(n, k) {
        Option.none {
            fin_value_or_zero[A](n, fin_zero_function[A](n), k) = A.0
            fin_value_or_zero[A](n, fin_zero_function[A](n), k) = nat_zero_function[A](k)
        }
        Option.some(i) {
            fin_value_or_zero[A](n, fin_zero_function[A](n), k) = fin_zero_function[A](n, i)
            fin_value_or_zero[A](n, fin_zero_function[A](n), k) = nat_zero_function[A](k)
        }
    }
}

/// The finite sum of the zero function is zero.
theorem fin_sum_zero_function[A: AddCommMonoid](n: Nat) {
    fin_sum[A](n, fin_zero_function[A](n)) = A.0
} by {
    forall(k: Nat) {
        fin_value_or_zero_zero_function[A](n, k)
        fin_value_or_zero[A](n, fin_zero_function[A](n), k) = nat_zero_function[A](k)
    }
    function_extensionality(fin_value_or_zero[A](n, fin_zero_function[A](n)), nat_zero_function[A])
    partial_nat_zero_function[A](n)
}

/// The zero extension is zero outside the finite range.
theorem fin_value_or_zero_of_not_lt[A: Zero](n: Nat, f: Fin[n] -> A, k: Nat) {
    not k < n implies fin_value_or_zero[A](n, f, k) = A.0
} by {
    if not k < n {
        match fin_of_nat_option(n, k) {
            Option.none {
                fin_value_or_zero[A](n, f, k) = A.0
            }
            Option.some(i) {
                Fin[n].new(k) = Option.some(i)
                fin_new_some_imp_lt(n, k, i)
                false
            }
        }
    }
}
