from nat import Nat
from data.basic.set import Set, set_image, all_sets_subset_universal,
    set_image_universal_of_surjective
from data.cardinal.countable import is_countable, countable_has_sequence_of_nonempty,
    subset_of_countable_is_countable
from data.basic.functions import surjective_fn_has_preimage
from int import Int
from rat import int_zigzag, int_zigzag_surjective

/// The universal set of integers is countable.
theorem int_universal_set_is_countable {
    is_countable[Int](Set[Int].universal_set)
} by {
    is_countable[Int](Set[Int].universal_set) =
        (Set[Int].universal_set = Set[Int].empty_set or exists(f: Nat -> Int) {
            forall(x: Int) {
                Set[Int].universal_set.contains(x) implies exists(n: Nat) { f(n) = x }
            }
        })
    forall(x: Int) {
        if Set[Int].universal_set.contains(x) {
            int_zigzag_surjective
            surjective_fn_has_preimage(int_zigzag, x)
            exists(n: Nat) { int_zigzag(n) = x }
        }
    }
    exists(f: Nat -> Int) {
        forall(x: Int) {
            Set[Int].universal_set.contains(x) implies exists(n: Nat) { f(n) = x }
        }
    }
    is_countable[Int](Set[Int].universal_set)
}

/// Every set of integers is countable.
theorem int_subset_is_countable(s: Set[Int]) {
    is_countable[Int](s)
} by {
    all_sets_subset_universal(s)
    int_universal_set_is_countable
    subset_of_countable_is_countable(s, Set[Int].universal_set)
}

/// The range of the integer zig-zag enumeration is the universal integer set.
theorem int_range_zigzag_is_universal {
    set_image(Set[Nat].universal_set, int_zigzag) = Set[Int].universal_set
} by {
    int_zigzag_surjective
    set_image_universal_of_surjective(int_zigzag)
}

/// Every nonempty set of integers has a sequence covering its elements.
theorem int_countable_has_sequence(s: Set[Int]) {
    s != Set[Int].empty_set implies exists(f: Nat -> Int) {
        forall(x: Int) { s.contains(x) implies exists(n: Nat) { f(n) = x } }
    }
} by {
    if s != Set[Int].empty_set {
        int_subset_is_countable(s)
        countable_has_sequence_of_nonempty(s)
    }
}
