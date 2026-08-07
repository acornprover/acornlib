from int import Int

/// A common divisor of two integers divides their difference.
///
/// `src/int/` has divisibility transitivity and the gcd lemmas but no subtraction rule, so the
/// witness is constructed here.
theorem int_divides_sub(d: Int, x: Int, y: Int) {
    d.divides(x) and d.divides(y) implies d.divides(x - y)
} by {
    if d.divides(x) and d.divides(y) {
        let (a: Int) satisfy {
            a * d = x
        }
        let (b: Int) satisfy {
            b * d = y
        }
        (a - b) * d = a * d - b * d
        (a - b) * d = x - y
        exists(c: Int) {
            c * d = x - y
        }
        d.divides(x - y)
    }
}

/// Cancelling a shared minuend from a difference of differences.
///
/// Elementary, but the rearrangement does not fall out of the ring axioms in one step.
theorem int_sub_sub_cancel(x: Int, a: Int, b: Int) {
    (x - b) - (x - a) = a - b
} by {
    x - b = x + -b
    x - a = x + -a
    -(x + -a) = -x + a
    (x - b) - (x - a) = (x + -b) + -(x + -a)
    (x + -b) + (-x + a) = (x + -x) + (-b + a)
    x + -x = Int.0
    Int.0 + (-b + a) = -b + a
    -b + a = a + -b
    a + -b = a - b
    (x - b) - (x - a) = a - b
}
