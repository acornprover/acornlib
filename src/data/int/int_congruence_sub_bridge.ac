from int import Int
from nat import Nat
from data.int.int_congruence import int_congr_mod_add, int_congr_mod_neg, int_congr_mod_refl
numerals Int

/// Integer congruence modulo n is preserved by subtraction.
theorem int_congr_mod_sub(a: Int, b: Int, c: Int, d: Int, n: Nat) {
    a.congr_mod(c, n) and b.congr_mod(d, n) implies (a - b).congr_mod(c - d, n)
} by {
    if a.congr_mod(c, n) and b.congr_mod(d, n) {
        int_congr_mod_neg(b, d, n)
        (-b).congr_mod(-d, n)
        int_congr_mod_add(a, -b, c, -d, n)
        (a + -b).congr_mod(c + -d, n)
        a + -b = a - b
        c + -d = c - d
        (a - b).congr_mod(c - d, n)
    }
}

/// Replacing a left minuend by a congruent one preserves subtraction congruence.
theorem int_congr_mod_sub_left(a: Int, b: Int, c: Int, n: Nat) {
    a.congr_mod(b, n) implies (a - c).congr_mod(b - c, n)
} by {
    if a.congr_mod(b, n) {
        int_congr_mod_refl(c, n)
        int_congr_mod_sub(a, c, b, c, n)
    }
}

/// Replacing a right subtrahend by a congruent one preserves subtraction congruence.
theorem int_congr_mod_sub_right(a: Int, b: Int, c: Int, n: Nat) {
    a.congr_mod(b, n) implies (c - a).congr_mod(c - b, n)
} by {
    if a.congr_mod(b, n) {
        int_congr_mod_refl(c, n)
        int_congr_mod_sub(c, a, c, b, n)
    }
}
