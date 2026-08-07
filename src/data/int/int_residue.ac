from int import Int, abs, neg_or_pos
from nat import Nat
from int import mul_from_nat
from zmod import int_mod_rel
numerals Int

/// Every integer has a Nat representative under congruence modulo a positive n.
theorem int_has_nat_residue(x: Int, n: Nat) {
    n != Nat.0 implies exists(r: Nat) { int_mod_rel(n, x, Int.from_nat(r)) }
} by {
    if n != Nat.0 {
        let m: Nat = abs(x)
        if x.is_negative {
            // x = -Int.from_nat(m); pick r = n*m - m so m + r = n*m, divisible by n.
            Int.from_nat(m) = -x
            x = -Int.from_nat(m)
            let r: Nat = n * m - m
            m <= n * m
            m + r = n * m
            Int.from_nat(m + r) = Int.from_nat(n * m)
            mul_from_nat(n, m)
            Int.from_nat(n) * Int.from_nat(m) = Int.from_nat(n * m)
            Int.from_nat(m + r) = Int.from_nat(n) * Int.from_nat(m)
            Int.from_nat(m) + Int.from_nat(r) = Int.from_nat(m + r)
            Int.from_nat(m) + Int.from_nat(r) = Int.from_nat(n) * Int.from_nat(m)
            x - Int.from_nat(r) = -Int.from_nat(m) - Int.from_nat(r)
            -Int.from_nat(m) - Int.from_nat(r) = -(Int.from_nat(m) + Int.from_nat(r))
            x - Int.from_nat(r) = -(Int.from_nat(n) * Int.from_nat(m))
            -(Int.from_nat(n) * Int.from_nat(m)) = Int.from_nat(n) * (-Int.from_nat(m))
            x - Int.from_nat(r) = Int.from_nat(n) * (-Int.from_nat(m))
            Int.from_nat(n).divides(x - Int.from_nat(r))
            exists(r0: Nat) { int_mod_rel(n, x, Int.from_nat(r0)) }
        } else {
            // x is nonneg, so x = Int.from_nat(abs(x)). Take r = abs(x).
            neg_or_pos(x)
            let r: Nat = m
            Int.from_nat(r) = x
            x - Int.from_nat(r) = Int.0
            Int.from_nat(n).divides(Int.0)
            int_mod_rel(n, x, Int.from_nat(r))
            exists(r0: Nat) { int_mod_rel(n, x, Int.from_nat(r0)) }
        }
    }
}
