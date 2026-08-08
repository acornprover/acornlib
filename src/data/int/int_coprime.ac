from nat import Nat, divides_lte, lte_antisymm
from algebra.comm_monoid_rearrange import mul_swap_inner
from int import Int, abs, exp_one, exp_zero, exp_add, spans, spans_gcd, gcd_comm, gcd_one_left, gcd_one_right,
    divides_gcd, gcd_div_left, gcd_div_right, gcd_nonneg, div_imp_div_abs

numerals Int

/// True if two integers have no common factor beyond a unit.
///
/// Stated as the greatest common divisor being one rather than through a quantifier, which is
/// what the Bezout identity is indexed by.
define is_coprime(a: Int, b: Int) -> Bool {
    a.gcd(b) = Int.1
}

/// Coprimality is the gcd condition.
theorem is_coprime_apply(a: Int, b: Int) {
    is_coprime(a, b) implies a.gcd(b) = Int.1
} by {
    if is_coprime(a, b) {
        (is_coprime(a, b) = (a.gcd(b) = Int.1))
        a.gcd(b) = Int.1
    }
}

/// The gcd condition is coprimality.
theorem is_coprime_intro(a: Int, b: Int) {
    a.gcd(b) = Int.1 implies is_coprime(a, b)
} by {
    if a.gcd(b) = Int.1 {
        (is_coprime(a, b) = (a.gcd(b) = Int.1))
        is_coprime(a, b)
    }
}

/// Coprimality is symmetric.
theorem is_coprime_symm(a: Int, b: Int) {
    is_coprime(a, b) implies is_coprime(b, a)
} by {
    if is_coprime(a, b) {
        is_coprime_apply(a, b)
        a.gcd(b) = Int.1
        gcd_comm(a, b)
        (a.gcd(b) = b.gcd(a))
        b.gcd(a) = Int.1
        is_coprime_intro(b, a)
        is_coprime(b, a)
    }
}

/// Everything is coprime to one.
theorem is_coprime_one_right(a: Int) {
    is_coprime(a, Int.1)
} by {
    gcd_one_right(a)
    a.gcd(Int.1) = Int.1
    is_coprime_intro(a, Int.1)
    is_coprime(a, Int.1)
}

/// Coprime integers admit a Bezout combination of one.
///
/// The form every argument below runs on: coprimality is useful because it writes one as a
/// combination, and multiplying that combination by anything writes that thing as a combination
/// too.
theorem coprime_bezout(a: Int, b: Int) {
    is_coprime(a, b) implies exists(u: Int, v: Int) { u * a + v * b = Int.1 }
} by {
    if is_coprime(a, b) {
        is_coprime_apply(a, b)
        a.gcd(b) = Int.1
        spans_gcd(a, b)
        spans(a, b, a.gcd(b))
        spans(a, b, Int.1)
        (spans(a, b, Int.1) = exists(d: Int, e: Int) { d * a + e * b = Int.1 })
        exists(u: Int, v: Int) { u * a + v * b = Int.1 }
    }
}

/// A Bezout combination of one makes the pair coprime.
theorem coprime_of_bezout(a: Int, b: Int, u: Int, v: Int) {
    u * a + v * b = Int.1 implies is_coprime(a, b)
} by {
    if u * a + v * b = Int.1 {
        gcd_div_left(a, b)
        a.gcd(b).divides(a)
        gcd_div_right(a, b)
        a.gcd(b).divides(b)
        (a.gcd(b).divides(a) = exists(d: Int) { d * a.gcd(b) = a })
        let (p: Int) satisfy {
            p * a.gcd(b) = a
        }
        (a.gcd(b).divides(b) = exists(d: Int) { d * a.gcd(b) = b })
        let (q: Int) satisfy {
            q * a.gcd(b) = b
        }
        (u * (p * a.gcd(b)) + v * (q * a.gcd(b)) = (u * p + v * q) * a.gcd(b))
        ((u * p + v * q) * a.gcd(b) = Int.1)
        (a.gcd(b).divides(Int.1) = exists(d: Int) { d * a.gcd(b) = Int.1 })
        a.gcd(b).divides(Int.1)
        div_imp_div_abs(a.gcd(b), Int.1)
        abs(a.gcd(b)).divides(abs(Int.1))
        (abs(Int.1) = Nat.1)
        abs(a.gcd(b)).divides(Nat.1)
        divides_lte(abs(a.gcd(b)), Nat.1)
        (Nat.1 = Nat.0 or abs(a.gcd(b)) <= Nat.1)
        abs(a.gcd(b)) <= Nat.1
        if abs(a.gcd(b)) = Nat.0 {
            (Nat.0.divides(Nat.1) = exists(k: Nat) { Nat.0 * k = Nat.1 })
            let (r: Nat) satisfy {
                Nat.0 * r = Nat.1
            }
            (Nat.0 * r = Nat.0)
            false
        }
        abs(a.gcd(b)) != Nat.0
        Nat.0 < abs(a.gcd(b))
        (Nat.0.suc = Nat.1)
        Nat.1 <= abs(a.gcd(b))
        lte_antisymm(abs(a.gcd(b)), Nat.1)
        abs(a.gcd(b)) = Nat.1
        gcd_nonneg(a, b)
        not a.gcd(b).is_negative
        a.gcd(b) = Int.from_nat(abs(a.gcd(b)))
        (Int.from_nat(Nat.1) = Int.1)
        a.gcd(b) = Int.1
        is_coprime_intro(a, b)
        is_coprime(a, b)
    }
}

/// Gauss's lemma for divisibility.
///
/// If a divisor of a product is coprime to one factor, it divides the other. This is what makes
/// a coprimality hypothesis strip a factor from a divisibility, and it is the step every
/// rational-root argument turns on.
theorem divides_of_coprime(a: Int, x: Int, y: Int) {
    a.divides(x * y) and is_coprime(a, y) implies a.divides(x)
} by {
    if a.divides(x * y) and is_coprime(a, y) {
        coprime_bezout(a, y)
        exists(u: Int, v: Int) { u * a + v * y = Int.1 }
        let (s: Int, t: Int) satisfy {
            s * a + t * y = Int.1
        }
        (a.divides(x * y) = exists(d: Int) { d * a = x * y })
        let (w: Int) satisfy {
            w * a = x * y
        }
        (x * (s * a + t * y) = x)
        (x * (s * a + t * y) = x * (s * a) + x * (t * y))
        (x * (s * a) = (x * s) * a)
        (x * (t * y) = (x * t) * y)
        (x * t = t * x)
        ((t * x) * y = t * (x * y))
        (x * (t * y) = t * (x * y))
        (t * (x * y) = t * (w * a))
        (t * (w * a) = (t * w) * a)
        ((x * s) * a + (t * w) * a = (x * s + t * w) * a)
        ((x * s + t * w) * a = x)
        (a.divides(x) = exists(d: Int) { d * a = x })
        a.divides(x)
    }
}

/// Coprimality to each factor gives coprimality to the product.
///
/// The two Bezout combinations multiply: expanding the product leaves a multiple of `a` plus a
/// multiple of `x * y`, which is a combination of one for the pair.
theorem coprime_mul(a: Int, x: Int, y: Int) {
    is_coprime(a, x) and is_coprime(a, y) implies is_coprime(a, x * y)
} by {
    if is_coprime(a, x) and is_coprime(a, y) {
        coprime_bezout(a, x)
        exists(u: Int, v: Int) { u * a + v * x = Int.1 }
        let (s: Int, t: Int) satisfy {
            s * a + t * x = Int.1
        }
        coprime_bezout(a, y)
        exists(u: Int, v: Int) { u * a + v * y = Int.1 }
        let (p: Int, q: Int) satisfy {
            p * a + q * y = Int.1
        }
        (t * x = (t * x) * Int.1)
        (t * x = (t * x) * (p * a + q * y))
        ((t * x) * (p * a + q * y) = (t * x) * (p * a) + (t * x) * (q * y))
        mul_swap_inner[Int](t, x, p, a)
        ((t * x) * (p * a) = (t * p) * (x * a))
        ((t * p) * (x * a) = ((t * p) * x) * a)
        mul_swap_inner[Int](t, x, q, y)
        ((t * x) * (q * y) = (t * q) * (x * y))
        (t * x = ((t * p) * x) * a + (t * q) * (x * y))
        (Int.1 = s * a + (((t * p) * x) * a + (t * q) * (x * y)))
        (s * a + (((t * p) * x) * a + (t * q) * (x * y))
            = (s * a + ((t * p) * x) * a) + (t * q) * (x * y))
        (s * a + ((t * p) * x) * a = (s + (t * p) * x) * a)
        ((s + (t * p) * x) * a + (t * q) * (x * y) = Int.1)
        coprime_of_bezout(a, x * y, s + (t * p) * x, t * q)
        is_coprime(a, x * y)
    }
}

/// Coprimality passes to powers.
///
/// Induction on the exponent, with the zero case coprimality to one.
theorem coprime_pow(a: Int, b: Int, n: Nat) {
    is_coprime(a, b) implies is_coprime(a, b.pow(n))
} by {
    if is_coprime(a, b) {
        define w(k: Nat) -> Bool {
            is_coprime(a, b.pow(k))
        }
        exp_zero(b)
        (b.pow(Nat.0) = Int.1)
        is_coprime_one_right(a)
        is_coprime(a, Int.1)
        w(Nat.0)
        forall(k: Nat) {
            if w(k) {
                is_coprime(a, b.pow(k))
                coprime_mul(a, b.pow(k), b)
                is_coprime(a, b.pow(k) * b)
                exp_one(b)
                (b.pow(Nat.1) = b)
                exp_add(b, k, Nat.1)
                (b.pow(k + Nat.1) = b.pow(k) * b.pow(Nat.1))
                (b.pow(k + Nat.1) = b.pow(k) * b)
                (k + Nat.1 = k.suc)
                is_coprime(a, b.pow(k.suc))
                w(k.suc)
            }
            (w(k) implies w(k.suc))
        }
        w(Nat.0) and forall(k: Nat) {
            w(k) implies w(k.suc)
        }
        Nat.induction(w)
        w(n)
        is_coprime(a, b.pow(n))
    }
}

/// A divisor of a product with a coprime power factor divides the other factor.
///
/// The form the rational-root argument uses: the cleared equation carries a power of the
/// denominator, and coprimality strips the whole power at once.
theorem divides_of_coprime_pow(a: Int, x: Int, b: Int, n: Nat) {
    a.divides(x * b.pow(n)) and is_coprime(a, b) implies a.divides(x)
} by {
    if a.divides(x * b.pow(n)) and is_coprime(a, b) {
        coprime_pow(a, b, n)
        is_coprime(a, b.pow(n))
        divides_of_coprime(a, x, b.pow(n))
        a.divides(x)
    }
}
