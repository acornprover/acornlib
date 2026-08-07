/// Basic unit facts for integers.

from nat import Nat
from int import Int, abs, is_unit, two_units

numerals Int

/// An integer unit has absolute value one.
theorem int_unit_abs_eq_one(u: Int) {
    is_unit(u) implies abs(u) = Nat.1
} by {
    if is_unit(u) {
        is_unit(u) = (abs(u) = Nat.1)
    }
}

/// The integer unit predicate unfolds to absolute value one.
theorem is_unit_abs_eq_one(u: Int) {
    is_unit(u) implies abs(u) = Nat.1
} by {
    int_unit_abs_eq_one(u)
}

/// An integer unit is either one or negative one.
theorem is_unit_eq_one_or_neg_one(u: Int) {
    is_unit(u) implies u = 1 or u = -1
} by {
    two_units(u)
}

/// Either one or negative one is an integer unit.
theorem is_unit_of_eq_one_or_neg_one(u: Int) {
    u = 1 or u = -1 implies is_unit(u)
} by {
    if u = 1 or u = -1 {
        if u = 1 {
            abs(1) = Nat.1
            abs(u) = Nat.1
            is_unit(u) = (abs(u) = Nat.1)
            is_unit(u)
        } else {
            u = -1
            abs(-1) = Nat.1
            abs(u) = Nat.1
            is_unit(u) = (abs(u) = Nat.1)
            is_unit(u)
        }
    }
}

/// Integer units are exactly one and negative one.
theorem is_unit_iff(u: Int) {
    is_unit(u) = (u = 1 or u = -1)
} by {
    if is_unit(u) {
        is_unit_eq_one_or_neg_one(u)
        u = 1 or u = -1
        is_unit(u) = (u = 1 or u = -1)
    } else {
        if u = 1 or u = -1 {
            is_unit_of_eq_one_or_neg_one(u)
            false
        }
        is_unit(u) = (u = 1 or u = -1)
    }
}
