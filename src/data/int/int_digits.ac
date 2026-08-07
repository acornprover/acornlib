from int import Int, add_assoc, add_zero_left, add_zero_right
from list import List
from nat import Nat
from data.basic.relation_basic import equivalence_flip, equivalence_step, is_transitive
from zmod import int_mod_rel, int_mod_rel_add_compatible, int_mod_rel_is_equivalence,
    int_mod_rel_zero_iff_int_three_divides, int_read_mod3_compatible, int_read_mod3_step

/// Reads a list of base-10 integer digits from left to right, starting with an accumulator.
define int_read_digits_acc(digits: List[Int], acc: Int) -> Int {
    match digits {
        List.nil {
            acc
        }
        List.cons(head, tail) {
            int_read_digits_acc(tail, acc.read(head))
        }
    }
}

/// Reads a list of base-10 integer digits from left to right, starting at zero.
define int_read_digits(digits: List[Int]) -> Int {
    int_read_digits_acc(digits, Int.0)
}

/// The sum of a list of integer digits.
define int_digit_sum(digits: List[Int]) -> Int {
    match digits {
        List.nil {
            Int.0
        }
        List.cons(head, tail) {
            head + int_digit_sum(tail)
        }
    }
}

/// Reading no digits returns the accumulator.
theorem int_read_digits_acc_nil(acc: Int) {
    int_read_digits_acc(List.nil[Int], acc) = acc
}

/// Reading a cons-list performs one decimal step and then reads the tail.
theorem int_read_digits_acc_cons(head: Int, tail: List[Int], acc: Int) {
    int_read_digits_acc(List.cons(head, tail), acc) = int_read_digits_acc(tail, acc.read(head))
}

/// The zero-starting digit reader is the accumulator reader at zero.
theorem int_read_digits_eq_acc_zero(digits: List[Int]) {
    int_read_digits(digits) = int_read_digits_acc(digits, Int.0)
}

/// Reading no digits from zero gives zero.
theorem int_read_digits_nil {
    int_read_digits(List.nil[Int]) = Int.0
}

/// Reading a cons-list from zero performs one decimal step and then reads the tail.
theorem int_read_digits_cons(head: Int, tail: List[Int]) {
    int_read_digits(List.cons(head, tail)) = int_read_digits_acc(tail, Int.0.read(head))
}

/// Reading a single digit from an arbitrary accumulator is one decimal step.
theorem int_read_digits_acc_single(acc: Int, digit: Int) {
    int_read_digits_acc(List.cons(digit, List.nil[Int]), acc) = acc.read(digit)
}

/// Reading a single digit from zero is one decimal step from zero.
theorem int_read_digits_single(digit: Int) {
    int_read_digits(List.cons(digit, List.nil[Int])) = Int.0.read(digit)
}

/// The digit sum of the empty list is zero.
theorem int_digit_sum_nil {
    int_digit_sum(List.nil[Int]) = Int.0
}

/// The digit sum of a cons-list is the head plus the digit sum of the tail.
theorem int_digit_sum_cons(head: Int, tail: List[Int]) {
    int_digit_sum(List.cons(head, tail)) = head + int_digit_sum(tail)
}

/// The digit sum of a singleton list is the digit itself.
theorem int_digit_sum_single(digit: Int) {
    int_digit_sum(List.cons(digit, List.nil[Int])) = digit
} by {
    add_zero_right(digit)
}

/// The digit sum of a two-element list is the sum of its entries.
theorem int_digit_sum_pair(a: Int, b: Int) {
    int_digit_sum(List.cons(a, List.cons(b, List.nil[Int]))) = a + b
} by {
    int_digit_sum_single(b)
}

/// Reading the same list of digits preserves congruence modulo 3.
theorem int_read_digits_acc_mod3_compatible(digits: List[Int], x: Int, y: Int) {
    int_mod_rel(Nat.3, x, y) implies
    int_mod_rel(Nat.3, int_read_digits_acc(digits, x), int_read_digits_acc(digits, y))
} by {
    define p(ds: List[Int]) -> Bool {
        forall(a: Int, b: Int) {
            int_mod_rel(Nat.3, a, b) implies
            int_mod_rel(Nat.3, int_read_digits_acc(ds, a), int_read_digits_acc(ds, b))
        }
    }
    p(List.nil[Int])
    forall(head: Int, tail: List[Int]) {
        if p(tail) {
            forall(a: Int, b: Int) {
                if int_mod_rel(Nat.3, a, b) {
                    int_read_mod3_compatible(a, b, head)
                    int_mod_rel(Nat.3, int_read_digits_acc(tail, a.read(head)), int_read_digits_acc(tail, b.read(head)))
                    int_mod_rel(Nat.3, int_read_digits_acc(List.cons(head, tail), a), int_read_digits_acc(List.cons(head, tail), b))
                }
            }
            p(List.cons(head, tail))
        }
    }
    List.induction(function(ds: List[Int]) { p(ds) })
    p(digits)
}

/// Reading a digit list from congruent accumulators gives congruent final values.
theorem int_read_digits_acc_mod3_compatible_pair(digits: List[Int], x: Int, y: Int) {
    int_mod_rel(Nat.3, x, y) implies
    int_mod_rel(Nat.3, int_read_digits_acc(digits, x), int_read_digits_acc(digits, y))
} by {
    int_read_digits_acc_mod3_compatible(digits, x, y)
}

/// Reading a digit list from zero is congruent to reading it from any accumulator congruent to zero.
theorem int_read_digits_acc_mod3_of_acc_zero(digits: List[Int], acc: Int) {
    int_mod_rel(Nat.3, acc, Int.0) implies
    int_mod_rel(Nat.3, int_read_digits_acc(digits, acc), int_read_digits(digits))
} by {
    if int_mod_rel(Nat.3, acc, Int.0) {
        int_read_digits_acc_mod3_compatible(digits, acc, Int.0)
        int_mod_rel(Nat.3, int_read_digits_acc(digits, acc), int_read_digits(digits))
    }
}

/// A decimal digit-list read is congruent modulo 3 to the accumulator plus the digit sum.
theorem int_read_digits_acc_mod3_digit_sum(digits: List[Int], acc: Int) {
    int_mod_rel(Nat.3, int_read_digits_acc(digits, acc), acc + int_digit_sum(digits))
} by {
    define p(ds: List[Int]) -> Bool {
        forall(a: Int) {
            int_mod_rel(Nat.3, int_read_digits_acc(ds, a), a + int_digit_sum(ds))
        }
    }

    forall(a: Int) {
        add_zero_right(a)
        int_mod_rel_is_equivalence(Nat.3)
        int_mod_rel(Nat.3, a, a)
        int_mod_rel(Nat.3, int_read_digits_acc(List.nil[Int], a), a + int_digit_sum(List.nil[Int]))
    }
    p(List.nil[Int])

    forall(head: Int, tail: List[Int]) {
        if p(tail) {
            forall(a: Int) {
                let left = int_read_digits_acc(tail, a.read(head))
                let mid = a.read(head) + int_digit_sum(tail)
                let right = (a + head) + int_digit_sum(tail)

                function(x: Int) {
                    int_mod_rel(Nat.3, int_read_digits_acc(tail, x), x + int_digit_sum(tail))
                }(a.read(head))

                int_read_mod3_step(a, head)

                int_mod_rel_is_equivalence(Nat.3)
                is_transitive(int_mod_rel(Nat.3))
                int_mod_rel_add_compatible(Nat.3,
                    a.read(head), a + head,
                    int_digit_sum(tail), int_digit_sum(tail))
                int_mod_rel(Nat.3, mid, right)

                is_transitive(int_mod_rel(Nat.3)) = forall(x: Int, y: Int, z: Int) {
                    int_mod_rel(Nat.3, x, y) and int_mod_rel(Nat.3, y, z) implies
                    int_mod_rel(Nat.3, x, z)
                }
                function(x: Int, y: Int, z: Int) {
                    not int_mod_rel(Nat.3, x, y) or not int_mod_rel(Nat.3, y, z) or
                    int_mod_rel(Nat.3, x, z)
                }(left, mid, right)
                int_mod_rel(Nat.3, left, right)

                add_assoc(a, head, int_digit_sum(tail))
                int_mod_rel(Nat.3,
                    int_read_digits_acc(List.cons(head, tail), a),
                    a + int_digit_sum(List.cons(head, tail)))
            }
            p(List.cons(head, tail))
        }
    }

    List.induction(function(ds: List[Int]) { p(ds) })
    p(digits)
}

/// Reading a digit list from zero is congruent modulo 3 to its digit sum.
theorem int_read_digits_mod3_digit_sum(digits: List[Int]) {
    int_mod_rel(Nat.3, int_read_digits(digits), int_digit_sum(digits))
} by {
    int_read_digits_acc_mod3_digit_sum(digits, Int.0)
    add_zero_left(int_digit_sum(digits))
}

/// The divisibility-by-3 proposition for a read digit list agrees with the digit-sum proposition.
theorem int_read_digits_divisible_by_3_eq_digit_sum(digits: List[Int]) {
    Int.3.divides(int_read_digits(digits)) = Int.3.divides(int_digit_sum(digits))
} by {
    let n = int_read_digits(digits)
    let s = int_digit_sum(digits)

    int_read_digits_mod3_digit_sum(digits)

    int_mod_rel_is_equivalence(Nat.3)
    if int_mod_rel(Nat.3, n, Int.0) {
        equivalence_flip[Int](int_mod_rel(Nat.3), n, s)
        equivalence_step[Int](int_mod_rel(Nat.3), s, n, Int.0)
        int_mod_rel(Nat.3, s, Int.0)
    }
    if int_mod_rel(Nat.3, s, Int.0) {
        equivalence_step[Int](int_mod_rel(Nat.3), n, s, Int.0)
        int_mod_rel(Nat.3, n, Int.0)
    }
    int_mod_rel(Nat.3, n, Int.0) = int_mod_rel(Nat.3, s, Int.0)

    int_mod_rel_zero_iff_int_three_divides(n)
    int_mod_rel_zero_iff_int_three_divides(s)
}

/// Divisibility by 3 of the read value implies divisibility by 3 of the digit sum.
theorem int_read_digits_divisible_by_3_imp_digit_sum(digits: List[Int]) {
    Int.3.divides(int_read_digits(digits)) implies Int.3.divides(int_digit_sum(digits))
} by {
    if Int.3.divides(int_read_digits(digits)) {
        int_read_digits_divisible_by_3_eq_digit_sum(digits)
        Int.3.divides(int_digit_sum(digits))
    }
}

/// Divisibility by 3 of the digit sum implies divisibility by 3 of the read value.
theorem int_digit_sum_divisible_by_3_imp_read_digits(digits: List[Int]) {
    Int.3.divides(int_digit_sum(digits)) implies Int.3.divides(int_read_digits(digits))
} by {
    if Int.3.divides(int_digit_sum(digits)) {
        int_read_digits_divisible_by_3_eq_digit_sum(digits)
        Int.3.divides(int_read_digits(digits))
    }
}

/// The accumulator form of divisibility by 3 compares the read value with `acc + digit_sum`.
theorem int_read_digits_acc_divisible_by_3_eq_acc_plus_digit_sum(digits: List[Int], acc: Int) {
    Int.3.divides(int_read_digits_acc(digits, acc)) = Int.3.divides(acc + int_digit_sum(digits))
} by {
    let n = int_read_digits_acc(digits, acc)
    let s = acc + int_digit_sum(digits)

    int_read_digits_acc_mod3_digit_sum(digits, acc)

    int_mod_rel_is_equivalence(Nat.3)
    if int_mod_rel(Nat.3, n, Int.0) {
        equivalence_flip[Int](int_mod_rel(Nat.3), n, s)
        equivalence_step[Int](int_mod_rel(Nat.3), s, n, Int.0)
        int_mod_rel(Nat.3, s, Int.0)
    }
    if int_mod_rel(Nat.3, s, Int.0) {
        equivalence_step[Int](int_mod_rel(Nat.3), n, s, Int.0)
        int_mod_rel(Nat.3, n, Int.0)
    }
    int_mod_rel(Nat.3, n, Int.0) = int_mod_rel(Nat.3, s, Int.0)

    int_mod_rel_zero_iff_int_three_divides(n)
    int_mod_rel_zero_iff_int_three_divides(s)
}

/// The zero-accumulator read is congruent modulo 3 to the digit sum.
theorem int_read_digits_acc_zero_mod3_digit_sum(digits: List[Int]) {
    int_mod_rel(Nat.3, int_read_digits_acc(digits, Int.0), int_digit_sum(digits))
} by {
    int_read_digits_acc_mod3_digit_sum(digits, Int.0)
    add_zero_left(int_digit_sum(digits))
}

/// The zero-accumulator divisibility-by-3 proposition agrees with the digit-sum proposition.
theorem int_read_digits_acc_zero_divisible_by_3_eq_digit_sum(digits: List[Int]) {
    Int.3.divides(int_read_digits_acc(digits, Int.0)) = Int.3.divides(int_digit_sum(digits))
} by {
    int_read_digits_acc_divisible_by_3_eq_acc_plus_digit_sum(digits, Int.0)
    add_zero_left(int_digit_sum(digits))
}

/// The named zero-starting reader and the explicit zero accumulator have the same divisibility-by-3 proposition.
theorem int_read_digits_divisible_by_3_eq_acc_zero(digits: List[Int]) {
    Int.3.divides(int_read_digits(digits)) = Int.3.divides(int_read_digits_acc(digits, Int.0))
} by {
}
