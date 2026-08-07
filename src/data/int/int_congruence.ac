from int import Int
from nat import Nat
from zmod import int_mod_rel, int_mod_rel_is_equivalence,
    int_mod_rel_add_compatible, int_mod_rel_mul_compatible,
    int_mod_rel_neg_compatible, int_mod_rel_zero_eq, int_mod_rel_zero_iff_divides
from data.basic.relation_basic import is_reflexive, is_symmetric, is_transitive
numerals Int

attributes Int {
    /// True if this integer is congruent to b modulo n. Defined as the
    /// existing `int_mod_rel` divisibility-based predicate, with method
    /// syntax matching `Nat.congr_mod` so the uniform `a.congr_mod(b, n)`
    /// form works on both types.
    define congr_mod(self, b: Int, n: Nat) -> Bool {
        int_mod_rel(n, self, b)
    }
}

/// Method-style integer congruence is exactly the divisibility-based relation.
theorem int_congr_mod_iff_int_mod_rel(a: Int, b: Int, n: Nat) {
    a.congr_mod(b, n) = int_mod_rel(n, a, b)
} by {
}

/// Method-style congruence gives the underlying divisibility-based relation.
theorem int_mod_rel_of_congr_mod(a: Int, b: Int, n: Nat) {
    a.congr_mod(b, n) implies int_mod_rel(n, a, b)
} by {
    if a.congr_mod(b, n) {
        int_congr_mod_iff_int_mod_rel(a, b, n)
    }
}

/// The underlying divisibility-based relation gives method-style congruence.
theorem int_congr_mod_of_int_mod_rel(a: Int, b: Int, n: Nat) {
    int_mod_rel(n, a, b) implies a.congr_mod(b, n)
} by {
    if int_mod_rel(n, a, b) {
        int_congr_mod_iff_int_mod_rel(a, b, n)
    }
}

/// Congruence modulo zero is equality of integers in method-style form.
theorem int_congr_mod_zero_eq(a: Int, b: Int) {
    a.congr_mod(b, Nat.0) = (a = b)
} by {
    int_congr_mod_iff_int_mod_rel(a, b, Nat.0)
    int_mod_rel_zero_eq(a, b)
}

/// Method-style congruence to zero is divisibility by the modulus.
theorem int_congr_mod_zero_iff_divides(a: Int, n: Nat) {
    a.congr_mod(Int.0, n) = Int.from_nat(n).divides(a)
} by {
    int_congr_mod_iff_int_mod_rel(a, Int.0, n)
    int_mod_rel_zero_iff_divides(n, a)
}

/// Integer congruence modulo n is reflexive.
theorem int_congr_mod_refl(a: Int, n: Nat) {
    a.congr_mod(a, n)
} by {
    int_mod_rel_is_equivalence(n)
    is_reflexive(int_mod_rel(n))
}

/// Integer congruence modulo n is symmetric.
theorem int_congr_mod_symm(a: Int, b: Int, n: Nat) {
    a.congr_mod(b, n) implies b.congr_mod(a, n)
} by {
    int_mod_rel_is_equivalence(n)
    is_symmetric(int_mod_rel(n))
}

/// Integer congruence modulo n is transitive.
theorem int_congr_mod_trans(a: Int, b: Int, c: Int, n: Nat) {
    a.congr_mod(b, n) and b.congr_mod(c, n) implies a.congr_mod(c, n)
} by {
    if a.congr_mod(b, n) and b.congr_mod(c, n) {
        int_mod_rel(n, a, b)
        int_mod_rel(n, b, c)
        int_mod_rel_is_equivalence(n)
        is_transitive(int_mod_rel(n))
        int_mod_rel(n, a, c)
    }
}

/// Integer congruence modulo n is preserved by addition.
theorem int_congr_mod_add(a: Int, b: Int, c: Int, d: Int, n: Nat) {
    a.congr_mod(c, n) and b.congr_mod(d, n) implies (a + b).congr_mod(c + d, n)
} by {
    if a.congr_mod(c, n) and b.congr_mod(d, n) {
        int_mod_rel(n, b, d)
        int_mod_rel_add_compatible(n, a, c, b, d)
        int_mod_rel(n, a + b, c + d)
    }
}

/// Integer congruence modulo n is preserved by multiplication.
theorem int_congr_mod_mul(a: Int, b: Int, c: Int, d: Int, n: Nat) {
    a.congr_mod(c, n) and b.congr_mod(d, n) implies (a * b).congr_mod(c * d, n)
} by {
    if a.congr_mod(c, n) and b.congr_mod(d, n) {
        int_mod_rel(n, b, d)
        int_mod_rel_mul_compatible(n, a, c, b, d)
        int_mod_rel(n, a * b, c * d)
    }
}

/// Integer congruence modulo n is preserved by negation.
theorem int_congr_mod_neg(a: Int, b: Int, n: Nat) {
    a.congr_mod(b, n) implies (-a).congr_mod(-b, n)
} by {
    if a.congr_mod(b, n) {
        int_mod_rel_neg_compatible(n, a, b)
        int_mod_rel(n, -a, -b)
    }
}

/// Replacing a left addend by a congruent one preserves the sum congruence.
theorem int_congr_mod_add_left(a: Int, b: Int, c: Int, n: Nat) {
    a.congr_mod(b, n) implies (a + c).congr_mod(b + c, n)
} by {
    if a.congr_mod(b, n) {
        int_congr_mod_refl(c, n)
        int_congr_mod_add(a, c, b, c, n)
    }
}

/// Replacing a right addend by a congruent one preserves the sum congruence.
theorem int_congr_mod_add_right(a: Int, b: Int, c: Int, n: Nat) {
    a.congr_mod(b, n) implies (c + a).congr_mod(c + b, n)
} by {
    if a.congr_mod(b, n) {
        int_congr_mod_refl(c, n)
        int_congr_mod_add(c, a, c, b, n)
    }
}

/// Replacing a left factor by a congruent one preserves the product congruence.
theorem int_congr_mod_mul_left(a: Int, b: Int, c: Int, n: Nat) {
    a.congr_mod(b, n) implies (a * c).congr_mod(b * c, n)
} by {
    if a.congr_mod(b, n) {
        int_congr_mod_refl(c, n)
        int_congr_mod_mul(a, c, b, c, n)
    }
}

/// Replacing a right factor by a congruent one preserves the product congruence.
theorem int_congr_mod_mul_right(a: Int, b: Int, c: Int, n: Nat) {
    a.congr_mod(b, n) implies (c * a).congr_mod(c * b, n)
} by {
    if a.congr_mod(b, n) {
        int_congr_mod_refl(c, n)
        int_congr_mod_mul(c, a, c, b, n)
    }
}
