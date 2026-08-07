from list import List
from data.basic.set import Set, empty_set_contains_eq, union_contains_eq
from data.cardinal.countable import is_countable, empty_set_is_countable,
    union_of_countable_is_countable

/// The union of a finite list of sets.
define list_union_sets[T](items: List[Set[T]]) -> Set[T] {
    match items {
        List.nil {
            Set[T].empty_set
        }
        List.cons(head, tail) {
            head.union(list_union_sets(tail))
        }
    }
}

/// True when every set in a finite list is countable.
define list_sets_countable[T](items: List[Set[T]]) -> Bool {
    match items {
        List.nil {
            true
        }
        List.cons(head, tail) {
            is_countable[T](head) and list_sets_countable[T](tail)
        }
    }
}

/// The finite union of an empty list is empty.
theorem list_union_sets_nil[T] {
    list_union_sets[T](List.nil[Set[T]]) = Set[T].empty_set
}

/// The finite union of a cons list is the head union the finite union of the tail.
theorem list_union_sets_cons[T](head: Set[T], tail: List[Set[T]]) {
    list_union_sets[T](List.cons(head, tail)) = head.union(list_union_sets[T](tail))
}

/// No element belongs to the finite union of an empty list.
theorem list_union_sets_nil_contains_eq[T](x: T) {
    list_union_sets[T](List.nil[Set[T]]).contains(x) = false
} by {
    list_union_sets_nil[T]
    empty_set_contains_eq[T](x)
}

/// Membership in a cons finite union is membership in the head or in the tail union.
theorem list_union_sets_cons_contains_eq[T](head: Set[T], tail: List[Set[T]], x: T) {
    list_union_sets[T](List.cons(head, tail)).contains(x) =
        (head.contains(x) or list_union_sets[T](tail).contains(x))
} by {
    list_union_sets_cons[T](head, tail)
    union_contains_eq[T](head, list_union_sets[T](tail), x)
}

/// The empty list of sets is a list of countable sets.
theorem list_sets_countable_nil[T] {
    list_sets_countable[T](List.nil[Set[T]])
}

/// A cons list of countable sets has a countable head and a countable tail.
theorem list_sets_countable_cons_imp[T](head: Set[T], tail: List[Set[T]]) {
    list_sets_countable[T](List.cons(head, tail)) implies
        is_countable[T](head) and list_sets_countable[T](tail)
} by {
    if list_sets_countable[T](List.cons(head, tail)) {
        is_countable[T](head) and list_sets_countable[T](tail)
    }
}

/// A countable head and a countable tail build a cons list of countable sets.
theorem list_sets_countable_cons_intro[T](head: Set[T], tail: List[Set[T]]) {
    is_countable[T](head) and list_sets_countable[T](tail) implies
        list_sets_countable[T](List.cons(head, tail))
} by {
    if is_countable[T](head) and list_sets_countable[T](tail) {
        list_sets_countable[T](List.cons(head, tail))
    }
}

/// True when every member appearing in a finite list is countable.
define list_sets_all_countable[T](items: List[Set[T]]) -> Bool {
    forall(s: Set[T]) {
        items.contains(s) implies is_countable[T](s)
    }
}

/// An element of a list satisfying a pointwise predicate has that predicate.
lemma list_forall_contains_elim[A](items: List[A], pred: A -> Bool, item: A) {
    (forall(x: A) { items.contains(x) implies pred(x) }) and items.contains(item) implies pred(item)
}

/// The head belongs to its cons list.
lemma list_cons_contains_head[A](head: A, tail: List[A]) {
    List.cons(head, tail).contains(head)
}

/// Every tail member belongs to the corresponding cons list.
lemma list_cons_contains_tail[A](head: A, tail: List[A], item: A) {
    tail.contains(item) implies List.cons(head, tail).contains(item)
}

/// The empty finite list satisfies pointwise countability.
theorem list_sets_all_countable_nil[T] {
    list_sets_all_countable[T](List.nil[Set[T]])
}

/// Pointwise countability on a cons list makes the head countable.
theorem list_sets_all_countable_cons_head[T](head: Set[T], tail: List[Set[T]]) {
    list_sets_all_countable[T](List.cons(head, tail)) implies is_countable[T](head)
} by {
    define pred(s: Set[T]) -> Bool {
        is_countable[T](s)
    }
    if list_sets_all_countable[T](List.cons(head, tail)) {
        list_sets_all_countable[T](List.cons(head, tail)) =
            forall(s: Set[T]) { List.cons(head, tail).contains(s) implies is_countable[T](s) }
        list_cons_contains_head[Set[T]](head, tail)
        list_forall_contains_elim[Set[T]](List.cons(head, tail), pred, head)
        is_countable[T](head)
    }
}

/// Pointwise countability on a cons list restricts to the tail.
theorem list_sets_all_countable_cons_tail[T](head: Set[T], tail: List[Set[T]]) {
    list_sets_all_countable[T](List.cons(head, tail)) implies list_sets_all_countable[T](tail)
} by {
    if list_sets_all_countable[T](List.cons(head, tail)) {
        list_sets_all_countable[T](List.cons(head, tail)) =
            forall(s: Set[T]) { List.cons(head, tail).contains(s) implies is_countable[T](s) }
        forall(s: Set[T]) {
            if tail.contains(s) {
                list_cons_contains_tail[Set[T]](head, tail, s)
                is_countable[T](s)
            }
        }
        list_sets_all_countable[T](tail)
    }
}

/// A pointwise-countable cons list is recursively countable when its tail already is.
theorem list_sets_countable_cons_of_all_countable[T](head: Set[T], tail: List[Set[T]]) {
    list_sets_all_countable[T](List.cons(head, tail)) and
        list_sets_countable[T](tail) implies list_sets_countable[T](List.cons(head, tail))
} by {
    if list_sets_all_countable[T](List.cons(head, tail)) and list_sets_countable[T](tail) {
        list_sets_all_countable_cons_head[T](head, tail)
        list_sets_countable_cons_intro[T](head, tail)
        list_sets_countable[T](List.cons(head, tail))
    }
}

/// Pointwise countability of all listed members gives the recursive list predicate.
theorem list_sets_all_countable_implies_list_sets_countable[T](items: List[Set[T]]) {
    list_sets_all_countable[T](items) implies list_sets_countable[T](items)
} by {
    define p(xs: List[Set[T]]) -> Bool {
        list_sets_all_countable[T](xs) implies list_sets_countable[T](xs)
    }

    if list_sets_all_countable[T](List.nil[Set[T]]) {
        list_sets_countable_nil[T]
        list_sets_countable[T](List.nil[Set[T]])
    }
    p(List.nil[Set[T]])

    forall(head: Set[T], tail: List[Set[T]]) {
        if p(tail) {
            if list_sets_all_countable[T](List.cons(head, tail)) {
                list_sets_all_countable_cons_tail[T](head, tail)
                list_sets_countable_cons_of_all_countable[T](head, tail)
                list_sets_countable[T](List.cons(head, tail))
            }
            p(List.cons(head, tail))
        }
    }

    List.induction(function(xs: List[Set[T]]) { p(xs) })
    forall(xs: List[Set[T]]) {
        p(xs)
    }
}

/// Pointwise countability of all listed members gives the recursive list predicate.
theorem list_sets_countable_of_forall_contains[T](items: List[Set[T]]) {
    (forall(s: Set[T]) { items.contains(s) implies is_countable[T](s) }) implies
        list_sets_countable[T](items)
} by {
    if forall(s: Set[T]) { items.contains(s) implies is_countable[T](s) } {
        list_sets_all_countable[T](items)
        list_sets_all_countable_implies_list_sets_countable[T](items)
        list_sets_countable[T](items)
    }
}

/// The finite union of an empty list is countable.
theorem list_union_sets_nil_is_countable[T] {
    is_countable[T](list_union_sets[T](List.nil[Set[T]]))
} by {
    list_union_sets_nil[T]
    empty_set_is_countable[T]
}

/// Adding a countable head to a countable finite tail union gives a countable finite union.
theorem list_union_sets_cons_is_countable[T](head: Set[T], tail: List[Set[T]]) {
    is_countable[T](head) and is_countable[T](list_union_sets[T](tail)) implies
        is_countable[T](list_union_sets[T](List.cons(head, tail)))
} by {
    if is_countable[T](head) and is_countable[T](list_union_sets[T](tail)) {
        union_of_countable_is_countable[T](head, list_union_sets[T](tail))
        list_union_sets_cons[T](head, tail)
        is_countable[T](list_union_sets[T](List.cons(head, tail)))
    }
}

/// The union of a finite list of countable sets is countable.
theorem list_union_sets_is_countable[T](items: List[Set[T]]) {
    list_sets_countable[T](items) implies is_countable[T](list_union_sets[T](items))
} by {
    define p(xs: List[Set[T]]) -> Bool {
        list_sets_countable[T](xs) implies is_countable[T](list_union_sets[T](xs))
    }

    if list_sets_countable[T](List.nil[Set[T]]) {
        list_union_sets_nil_is_countable[T]
        is_countable[T](list_union_sets[T](List.nil[Set[T]]))
    }
    p(List.nil[Set[T]])

    forall(head: Set[T], tail: List[Set[T]]) {
        if p(tail) {
            if list_sets_countable[T](List.cons(head, tail)) {
                list_sets_countable_cons_imp[T](head, tail)
                list_union_sets_cons_is_countable[T](head, tail)
                is_countable[T](list_union_sets[T](List.cons(head, tail)))
            }
            p(List.cons(head, tail))
        }
    }

    List.induction(function(xs: List[Set[T]]) { p(xs) })
    p(items)
}

/// Pointwise countability of all listed sets makes their finite union countable.
theorem list_union_sets_of_forall_contains_is_countable[T](items: List[Set[T]]) {
    (forall(s: Set[T]) { items.contains(s) implies is_countable[T](s) }) implies
        is_countable[T](list_union_sets[T](items))
} by {
    if forall(s: Set[T]) { items.contains(s) implies is_countable[T](s) } {
        list_sets_countable_of_forall_contains[T](items)
        list_union_sets_is_countable[T](items)
        is_countable[T](list_union_sets[T](items))
    }
}

/// A member set contributes each of its elements to the finite list union.
theorem list_union_sets_contains_of_member_contains[T](items: List[Set[T]], member: Set[T], x: T) {
    items.contains(member) and member.contains(x) implies list_union_sets[T](items).contains(x)
} by {
    define p(xs: List[Set[T]]) -> Bool {
        forall(s: Set[T], y: T) {
            xs.contains(s) and s.contains(y) implies list_union_sets[T](xs).contains(y)
        }
    }

    forall(s: Set[T], y: T) {
        if List.nil[Set[T]].contains(s) and s.contains(y) {
            false
        }
    }
    p(List.nil[Set[T]])

    forall(head: Set[T], tail: List[Set[T]]) {
        if p(tail) {
            forall(s: Set[T], y: T) {
                if List.cons(head, tail).contains(s) and s.contains(y) {
                    if head = s {
                        list_union_sets_cons_contains_eq[T](head, tail, y)
                        list_union_sets[T](List.cons(head, tail)).contains(y)
                    } else {
                        tail.contains(s)
                        p(tail) = forall(d: Set[T], z: T) {
                            tail.contains(d) and d.contains(z) implies list_union_sets[T](tail).contains(z)
                        }
                        list_union_sets_cons_contains_eq[T](head, tail, y)
                        list_union_sets[T](List.cons(head, tail)).contains(y)
                    }
                }
            }
            p(List.cons(head, tail))
        }
    }

    List.induction(function(xs: List[Set[T]]) { p(xs) })
    p(items)
}
