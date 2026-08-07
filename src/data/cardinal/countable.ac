from nat import Nat
from data.basic.set import Set, set_image, image_contains, empty_set_contains_eq,
    set_eq_empty_of_is_empty, set_image_empty, union_contains_cases,
    union_with_empty_is_self, union_comm

/// True if `s` is empty or every element of `s` appears in some term of a
/// sequence indexed by the natural numbers.
define is_countable[T](s: Set[T]) -> Bool {
    s = Set[T].empty_set or exists(f: Nat -> T) {
        forall(x: T) {
            s.contains(x) implies exists(n: Nat) { f(n) = x }
        }
    }
}

/// The empty set is countable.
theorem empty_set_is_countable[T] {
    is_countable[T](Set[T].empty_set)
} by {
    Set[T].empty_set = Set[T].empty_set
}

/// A nonempty countable set is covered by a sequence.
theorem countable_has_sequence_of_nonempty[T](s: Set[T]) {
    is_countable[T](s) and s != Set[T].empty_set implies exists(f: Nat -> T) {
        forall(x: T) {
            s.contains(x) implies exists(n: Nat) { f(n) = x }
        }
    }
} by {
    if is_countable[T](s) and s != Set[T].empty_set {
        if s = Set[T].empty_set {
            false
        }
    }
}

/// A subset of a sequence-covered set is countable.
theorem subset_of_sequence_covered_is_countable[T](s: Set[T], t: Set[T], f: Nat -> T) {
    s.subset(t) and forall(x: T) {
        t.contains(x) implies exists(n: Nat) { f(n) = x }
    } implies is_countable[T](s)
} by {
    if s.subset(t) and forall(x: T) {
        t.contains(x) implies exists(n: Nat) { f(n) = x }
    } {
        is_countable[T](s) = (s = Set[T].empty_set or exists(g: Nat -> T) {
            forall(x: T) {
                s.contains(x) implies exists(n: Nat) { g(n) = x }
            }
        })
        forall(x: T) {
            if s.contains(x) {
                t.contains(x)
                exists(n: Nat) { f(n) = x }
            }
        }
        exists(g: Nat -> T) {
            forall(x: T) {
                s.contains(x) implies exists(n: Nat) { g(n) = x }
            }
        }
        is_countable[T](s)
    }
}

/// The image of a sequence-covered set is countable.
theorem image_of_sequence_covered_is_countable[T, U](s: Set[T], h: T -> U, f: Nat -> T) {
    (forall(x: T) {
        s.contains(x) implies exists(n: Nat) { f(n) = x }
    }) implies is_countable[U](set_image(s, h))
} by {
    if forall(x: T) {
        s.contains(x) implies exists(n: Nat) { f(n) = x }
    } {
        is_countable[U](set_image(s, h)) = (set_image(s, h) = Set[U].empty_set or exists(k: Nat -> U) {
            forall(y: U) {
                set_image(s, h).contains(y) implies exists(n: Nat) { k(n) = y }
            }
        })
        let g: Nat -> U = function(n: Nat) { h(f(n)) }
        forall(y: U) {
            if set_image(s, h).contains(y) {
                let x: T satisfy { s.contains(x) and y = h(x) }
                let n: Nat satisfy { f(n) = x }
                g(n) = y
                exists(m: Nat) { g(m) = y }
            }
        }
        exists(k: Nat -> U) {
            forall(y: U) {
                set_image(s, h).contains(y) implies exists(n: Nat) { k(n) = y }
            }
        }
        is_countable[U](set_image(s, h))
    }
}

/// A subset of the empty set is countable.
theorem subset_of_empty_is_countable[T](s: Set[T]) {
    s.subset(Set[T].empty_set) implies is_countable[T](s)
} by {
    if s.subset(Set[T].empty_set) {
        forall(x: T) {
            if s.contains(x) {
                Set[T].empty_set.contains(x)
                empty_set_contains_eq[T](x)
                false
            }
        }
        s.is_empty
        set_eq_empty_of_is_empty(s)
        empty_set_is_countable[T]
        is_countable[T](s)
    }
}

/// A subset of a nonempty countable set is countable.
theorem subset_of_nonempty_countable_is_countable[T](s: Set[T], t: Set[T]) {
    s.subset(t) and is_countable[T](t) and t != Set[T].empty_set implies is_countable[T](s)
} by {
    if s.subset(t) and is_countable[T](t) and t != Set[T].empty_set {
        countable_has_sequence_of_nonempty(t)
        let f: Nat -> T satisfy {
            forall(x: T) {
                t.contains(x) implies exists(n: Nat) { f(n) = x }
            }
        }
        subset_of_sequence_covered_is_countable(s, t, f)
        is_countable[T](s)
    }
}

/// Every subset of a countable set is countable.
theorem subset_of_countable_is_countable[T](s: Set[T], t: Set[T]) {
    s.subset(t) and is_countable[T](t) implies is_countable[T](s)
} by {
    if s.subset(t) and is_countable[T](t) {
        if t = Set[T].empty_set {
            subset_of_empty_is_countable(s)
            is_countable[T](s)
        }
        if t != Set[T].empty_set {
            subset_of_nonempty_countable_is_countable(s, t)
            is_countable[T](s)
        }
    }
}

/// The image of a countable set is countable.
theorem image_of_countable_is_countable[T, U](s: Set[T], h: T -> U) {
    is_countable[T](s) implies is_countable[U](set_image(s, h))
} by {
    if is_countable[T](s) {
        if s = Set[T].empty_set {
            set_image_empty(h)
            empty_set_is_countable[U]
            is_countable[U](set_image(s, h))
        }
        if s != Set[T].empty_set {
            countable_has_sequence_of_nonempty(s)
            let f: Nat -> T satisfy {
                forall(x: T) {
                    s.contains(x) implies exists(n: Nat) { f(n) = x }
                }
            }
            image_of_sequence_covered_is_countable(s, h, f)
            is_countable[U](set_image(s, h))
        }
    }
}

/// The tail sequence obtained by dropping the first term.
define sequence_tail[T](f: Nat -> T, n: Nat) -> T {
    f(n.suc)
}

/// The even index corresponding to a natural number.
define countable_even_index(n: Nat) -> Nat {
    match n {
        Nat.zero {
            Nat.0
        }
        Nat.suc(pred) {
            countable_even_index(pred).suc.suc
        }
    }
}

/// The odd index corresponding to a natural number.
define countable_odd_index(n: Nat) -> Nat {
    countable_even_index(n).suc
}

/// The sequence formed by alternating terms of two sequences.
define interleave_sequences[T](f: Nat -> T, g: Nat -> T, n: Nat) -> T {
    match n {
        Nat.zero {
            f(Nat.0)
        }
        Nat.suc(pred) {
            interleave_sequences(g, sequence_tail(f), pred)
        }
    }
}

/// Dropping two interleaved positions advances both source sequences.
theorem interleave_sequences_suc_suc[T](f: Nat -> T, g: Nat -> T, n: Nat) {
    interleave_sequences(f, g, n.suc.suc) =
        interleave_sequences(sequence_tail(f), sequence_tail(g), n)
}

/// Every term of the left sequence occurs at its even index in the interleaving.
theorem interleave_sequences_left_index[T](f: Nat -> T, g: Nat -> T, n: Nat) {
    interleave_sequences(f, g, countable_even_index(n)) = f(n)
} by {
    define p(k: Nat) -> Bool {
        forall(a: Nat -> T, b: Nat -> T) {
            interleave_sequences(a, b, countable_even_index(k)) = a(k)
        }
    }
    forall(a: Nat -> T, b: Nat -> T) {
        countable_even_index(Nat.0) = Nat.0
        interleave_sequences(a, b, countable_even_index(Nat.0)) = a(Nat.0)
    }
    p(Nat.0)
    forall(k: Nat) {
        if p(k) {
            forall(a: Nat -> T, b: Nat -> T) {
                let atail: Nat -> T = sequence_tail(a)
                let btail: Nat -> T = sequence_tail(b)
                interleave_sequences(atail, btail, countable_even_index(k)) = atail(k)
                interleave_sequences_suc_suc(a, b, countable_even_index(k))
                interleave_sequences(a, b, countable_even_index(k.suc)) = a(k.suc)
            }
            p(k.suc)
        }
    }
    p(n)
}

/// Every term of the right sequence occurs at its odd index in the interleaving.
theorem interleave_sequences_right_index[T](f: Nat -> T, g: Nat -> T, n: Nat) {
    interleave_sequences(f, g, countable_odd_index(n)) = g(n)
} by {
    define p(k: Nat) -> Bool {
        forall(a: Nat -> T, b: Nat -> T) {
            interleave_sequences(a, b, countable_odd_index(k)) = b(k)
        }
    }
    forall(a: Nat -> T, b: Nat -> T) {
        countable_odd_index(Nat.0) = countable_even_index(Nat.0).suc
        countable_even_index(Nat.0) = Nat.0
        interleave_sequences(a, b, countable_odd_index(Nat.0)) = b(Nat.0)
    }
    p(Nat.0)
    forall(k: Nat) {
        if p(k) {
            forall(a: Nat -> T, b: Nat -> T) {
                let atail: Nat -> T = sequence_tail(a)
                let btail: Nat -> T = sequence_tail(b)
                interleave_sequences_suc_suc(a, b, countable_odd_index(k))
                interleave_sequences(a, b, countable_odd_index(k.suc)) = b(k.suc)
            }
            p(k.suc)
        }
    }
    p(n)
}

/// The union of two countable sets is countable.
theorem union_of_countable_is_countable[T](s: Set[T], t: Set[T]) {
    is_countable[T](s) and is_countable[T](t) implies is_countable[T](s.union(t))
} by {
    if is_countable[T](s) and is_countable[T](t) {
        if s = Set[T].empty_set {
            union_comm(s, t)
            union_with_empty_is_self(t)
            is_countable[T](s.union(t))
        } else {
            if t = Set[T].empty_set {
                union_with_empty_is_self(s)
                is_countable[T](s.union(t))
            } else {
                countable_has_sequence_of_nonempty(s)
                countable_has_sequence_of_nonempty(t)
                let f: Nat -> T satisfy {
                    forall(x: T) {
                        s.contains(x) implies exists(n: Nat) { f(n) = x }
                    }
                }
                let g: Nat -> T satisfy {
                    forall(x: T) {
                        t.contains(x) implies exists(n: Nat) { g(n) = x }
                    }
                }
                let h: Nat -> T = interleave_sequences(f, g)
                is_countable[T](s.union(t)) = (s.union(t) = Set[T].empty_set or exists(q: Nat -> T) {
                    forall(x: T) {
                        s.union(t).contains(x) implies exists(n: Nat) { q(n) = x }
                    }
                })
                forall(x: T) {
                    if s.union(t).contains(x) {
                        union_contains_cases(s, t, x)
                        if s.contains(x) {
                            let n: Nat satisfy { f(n) = x }
                            interleave_sequences_left_index(f, g, n)
                            h(countable_even_index(n)) = x
                            exists(k: Nat) { h(k) = x }
                        } else {
                            let n: Nat satisfy { g(n) = x }
                            interleave_sequences_right_index(f, g, n)
                            h(countable_odd_index(n)) = x
                            exists(k: Nat) { h(k) = x }
                        }
                    }
                }
                exists(q: Nat -> T) {
                    forall(x: T) {
                        s.union(t).contains(x) implies exists(n: Nat) { q(n) = x }
                    }
                }
                is_countable[T](s.union(t))
            }
        }
    }
}

/// The universal set on `Nat` is countable.
theorem nat_universal_is_countable {
    is_countable[Nat](Set[Nat].universal_set)
} by {
    let f: Nat -> Nat = function(n: Nat) { n }
    forall(x: Nat) {
        if Set[Nat].universal_set.contains(x) {
            exists(n: Nat) { f(n) = x }
        }
    }
}
