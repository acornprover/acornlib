from nat import Nat
from list import List, range_contains_iff_lt
from data.basic.set import Set, finite_constraint, list_set, finite_set_has_list_set, list_set_contains_eq,
    list_indexed_union, list_indexed_union_contains_eq,
    list_indexed_union_contains_witness, range_indexed_union, set_image, singleton_contains_eq,
    insert_eq_union_singleton, set_ext, union_contains_eq, empty_set_contains_eq
from data.cardinal.countable import is_countable, empty_set_is_countable,
    image_of_countable_is_countable, union_of_countable_is_countable

/// A singleton set is countable.
theorem singleton_set_is_countable[T](x: T) {
    is_countable[T](Set[T].singleton(x))
} by {
    let f: Nat -> T = function(n: Nat) { x }
    is_countable[T](Set[T].singleton(x)) = (Set[T].singleton(x) = Set[T].empty_set or exists(g: Nat -> T) {
        forall(y: T) {
            Set[T].singleton(x).contains(y) implies exists(n: Nat) { g(n) = y }
        }
    })
    forall(y: T) {
        if Set[T].singleton(x).contains(y) {
            singleton_contains_eq[T](x, y)
            exists(n: Nat) { f(n) = y }
        }
    }
    exists(g: Nat -> T) {
        forall(y: T) {
            Set[T].singleton(x).contains(y) implies exists(n: Nat) { g(n) = y }
        }
    }
}

/// Inserting one element into a countable set gives a countable set.
theorem insert_countable_is_countable[T](s: Set[T], item: T) {
    is_countable[T](s) implies is_countable[T](s.insert(item))
} by {
    if is_countable[T](s) {
        singleton_set_is_countable[T](item)
        union_of_countable_is_countable[T](s, Set[T].singleton(item))
        insert_eq_union_singleton[T](s, item)
        is_countable[T](s.insert(item))
    }
}

/// The set associated to a cons list is the union of the head singleton and the tail set.
theorem list_set_cons_eq_union_singleton[T](head: T, tail: List[T]) {
    list_set(List.cons(head, tail)) = Set[T].singleton(head).union(list_set(tail))
} by {
    forall(x: T) {
        list_set_contains_eq(List.cons(head, tail), x)
        list_set_contains_eq(tail, x)
        singleton_contains_eq[T](head, x)
        union_contains_eq(Set[T].singleton(head), list_set(tail), x)
        if list_set(List.cons(head, tail)).contains(x) {
            if head = x {
                Set[T].singleton(head).union(list_set(tail)).contains(x)
            } else {
                tail.contains(x)
                Set[T].singleton(head).union(list_set(tail)).contains(x)
            }
        }
        if Set[T].singleton(head).union(list_set(tail)).contains(x) {
            if Set[T].singleton(head).contains(x) {
                list_set(List.cons(head, tail)).contains(x)
            } else {
                list_set(List.cons(head, tail)).contains(x)
            }
        }
        list_set(List.cons(head, tail)).contains(x) =
            Set[T].singleton(head).union(list_set(tail)).contains(x)
    }
    set_ext(list_set(List.cons(head, tail)), Set[T].singleton(head).union(list_set(tail)))
}

/// A cons list has the set given by the union of the singleton head and tail set.
theorem list_set_cons_eq_singleton_union[T](head: T, tail: List[T]) {
    list_set[T](List.cons(head, tail)) = Set[T].singleton(head).union(list_set[T](tail))
} by {
    list_set_cons_eq_union_singleton[T](head, tail)
}

/// Every set associated to a list is countable.
theorem list_set_is_countable[T](items: List[T]) {
    is_countable[T](list_set(items))
} by {
    define p(xs: List[T]) -> Bool {
        is_countable[T](list_set(xs))
    }

    forall(x: T) {
        list_set_contains_eq(List.nil[T], x)
        empty_set_contains_eq[T](x)
        list_set(List.nil[T]).contains(x) = Set[T].empty_set.contains(x)
    }
    set_ext(list_set(List.nil[T]), Set[T].empty_set)
    empty_set_is_countable[T]
    p(List.nil[T])

    forall(head: T, tail: List[T]) {
        if p(tail) {
            singleton_set_is_countable(head)
            list_set_cons_eq_union_singleton(head, tail)
            union_of_countable_is_countable(Set[T].singleton(head), list_set(tail))
            is_countable[T](Set[T].singleton(head).union(list_set(tail)))
            p(List.cons(head, tail))
        }
    }

    List.induction(function(xs: List[T]) { p(xs) })
    p(items)
}

lemma list_indexed_union_nil_eq_empty[K, I](family: I -> Set[K]) {
    list_indexed_union(List.nil[I], family) = Set[K].empty_set
} by {
    forall(x: K) {
        list_indexed_union_contains_eq(List.nil[I], family, x)
        empty_set_contains_eq[K](x)
        if list_indexed_union(List.nil[I], family).contains(x) {
            let i: I satisfy {
                List.nil[I].contains(i) and family(i).contains(x)
            }
            false
        }
        list_indexed_union(List.nil[I], family).contains(x) = Set[K].empty_set.contains(x)
    }
    set_ext(list_indexed_union(List.nil[I], family), Set[K].empty_set)
}

lemma list_indexed_union_cons_eq_union[K, I](head: I, tail: List[I], family: I -> Set[K]) {
    list_indexed_union(List.cons(head, tail), family) =
        family(head).union(list_indexed_union(tail, family))
} by {
    forall(x: K) {
        list_indexed_union_contains_eq(List.cons(head, tail), family, x)
        list_indexed_union_contains_eq(tail, family, x)
        union_contains_eq(family(head), list_indexed_union(tail, family), x)
        if list_indexed_union(List.cons(head, tail), family).contains(x) {
            let i: I satisfy {
                List.cons(head, tail).contains(i) and family(i).contains(x)
            }
            if head = i {
                family(head).union(list_indexed_union(tail, family)).contains(x)
            } else {
                tail.contains(i)
                family(head).union(list_indexed_union(tail, family)).contains(x)
            }
        }
        if family(head).union(list_indexed_union(tail, family)).contains(x) {
            if family(head).contains(x) {
                list_indexed_union(List.cons(head, tail), family).contains(x)
            } else {
                list_indexed_union_contains_witness(tail, family, x)
                let i: I satisfy {
                    tail.contains(i) and family(i).contains(x)
                }
                list_indexed_union(List.cons(head, tail), family).contains(x)
            }
        }
        list_indexed_union(List.cons(head, tail), family).contains(x) =
            family(head).union(list_indexed_union(tail, family)).contains(x)
    }
    set_ext(list_indexed_union(List.cons(head, tail), family),
        family(head).union(list_indexed_union(tail, family)))
}

/// A list-indexed union of countable sets is countable.
theorem list_indexed_union_of_countable_is_countable[K, I](items: List[I], family: I -> Set[K]) {
    (forall(i: I) { items.contains(i) implies is_countable[K](family(i)) }) implies
    is_countable[K](list_indexed_union(items, family))
} by {
    define p(xs: List[I]) -> Bool {
        (forall(i: I) { xs.contains(i) implies is_countable[K](family(i)) }) implies
        is_countable[K](list_indexed_union(xs, family))
    }

    if forall(i: I) { List.nil[I].contains(i) implies is_countable[K](family(i)) } {
        list_indexed_union_nil_eq_empty[K, I](family)
        empty_set_is_countable[K]
        is_countable[K](list_indexed_union(List.nil[I], family))
    }
    p(List.nil[I])

    forall(head: I, tail: List[I]) {
        if p(tail) {
            if forall(i: I) { List.cons(head, tail).contains(i) implies is_countable[K](family(i)) } {
                List.cons(head, tail).contains(head)
                is_countable[K](family(head))
                forall(i: I) {
                    if tail.contains(i) {
                        List.cons(head, tail).contains(i)
                        is_countable[K](family(i))
                    }
                }
                p(tail)
                is_countable[K](list_indexed_union(tail, family))
                union_of_countable_is_countable(family(head), list_indexed_union(tail, family))
                list_indexed_union_cons_eq_union(head, tail, family)
                is_countable[K](list_indexed_union(List.cons(head, tail), family))
            }
            p(List.cons(head, tail))
        }
    }

    List.induction(function(xs: List[I]) { p(xs) })
    forall(xs: List[I]) { p(xs) }
    p(items)
}

/// A union indexed by an initial Nat range of countable sets is countable.
theorem range_indexed_union_of_countable_is_countable[K](n: Nat, family: Nat -> Set[K]) {
    (forall(i: Nat) { i < n implies is_countable[K](family(i)) }) implies
    is_countable[K](range_indexed_union(n, family))
} by {
    if forall(i: Nat) { i < n implies is_countable[K](family(i)) } {
        forall(i: Nat) {
            if n.range.contains(i) {
                range_contains_iff_lt(n, i)
                is_countable[K](family(i))
            }
        }
        list_indexed_union_of_countable_is_countable(n.range, family)
        is_countable[K](list_indexed_union(n.range, family))
        is_countable[K](range_indexed_union(n, family))
    }
}

/// Every finite set is countable.
theorem finite_set_is_countable[T](s: Set[T]) {
    s.is_finite implies is_countable[T](s)
} by {
    if s.is_finite {
        finite_set_has_list_set(s)
        let items: List[T] satisfy {
            list_set(items) = s
        }
        list_set_is_countable(items)
        is_countable[T](s)
    }
}

/// The image of a finite set is countable.
theorem finite_set_image_is_countable[T, U](s: Set[T], h: T -> U) {
    s.is_finite implies is_countable[U](set_image(s, h))
} by {
    if s.is_finite {
        finite_set_is_countable(s)
        image_of_countable_is_countable(s, h)
        is_countable[U](set_image(s, h))
    }
}

/// A finite index set has a list-indexed countable union over any countable family on that set.
theorem finite_set_has_countable_list_indexed_union[K, I](indices: Set[I], family: I -> Set[K]) {
    indices.is_finite and
    (forall(i: I) { indices.contains(i) implies is_countable[K](family(i)) }) implies
    exists(items: List[I]) {
        list_set(items) = indices and is_countable[K](list_indexed_union(items, family))
    }
} by {
    if indices.is_finite and
        (forall(i: I) { indices.contains(i) implies is_countable[K](family(i)) }) {
        finite_set_has_list_set(indices)
        let items: List[I] satisfy {
            list_set(items) = indices
        }
        forall(i: I) {
            if items.contains(i) {
                list_set_contains_eq(items, i)
                is_countable[K](family(i))
            }
        }
        list_indexed_union_of_countable_is_countable(items, family)
        exists(result: List[I]) {
            list_set(result) = indices and is_countable[K](list_indexed_union(result, family))
        }
    }
}

/// A predicate satisfying the finite constraint gives a countable set.
theorem finite_constraint_set_is_countable[T](contains: T -> Bool) {
    finite_constraint[T](contains) implies is_countable[T](Set[T].new(contains))
} by {
    if finite_constraint[T](contains) {
        Set[T].new(contains).contains = contains
        Set[T].new(contains).is_finite
        finite_set_is_countable[T](Set[T].new(contains))
        is_countable[T](Set[T].new(contains))
    }
}
