from nat import Nat
from pair import Pair, pair_eta, pair_new_first, pair_new_second
from sum_type import Sum
from rat import nat_pair_zigzag, nat_pair_zigzag_surjective
from data.basic.functions import is_bijection_fn, bijection_fn_is_surjective,
    surjective_fn_has_preimage
from data.basic.logic import exists_intro
from data.basic.set import Set, set_ext, set_image, set_image_empty,
    set_image_contains_witness, set_image_universal_of_surjective,
    universal_set_contains_eq, empty_set_contains_eq, set_eq_contains_at,
    maps_into_set_image
from data.basic.pair_set import pair_set, pair_set_contains_eq, pair_set_contains_imp_left,
    pair_set_contains_imp_right
from data.basic.sum_set import sum_set, sum_set_contains_inl, sum_set_contains_inr
from data.cardinal.countable import is_countable, empty_set_is_countable,
    countable_has_sequence_of_nonempty, countable_even_index, countable_odd_index,
    interleave_sequences, interleave_sequences_left_index, interleave_sequences_right_index,
    nat_universal_is_countable, image_of_countable_is_countable,
    subset_of_countable_is_countable
from list import List

/// The image of a countable set under a function is countable.
///
/// This is the transfer lemma of cardinal arithmetic: a bijection between two
/// sets carries countability from one to the other. Bijectivity is not needed
/// for the image alone, but the statement is stated with the bijection because
/// that is the form the transfer lemma takes.
theorem countable_transfer_along_bijection[T, U](s: Set[T], f: T -> U) {
    is_countable[T](s) and is_bijection_fn(f) implies is_countable[U](set_image(s, f))
} by {
    if is_countable[T](s) and is_bijection_fn(f) {
        if s = Set[T].empty_set {
            set_image_empty(f)
            empty_set_is_countable[U]
            is_countable[U](set_image(s, f))
        }
        if s != Set[T].empty_set {
            countable_has_sequence_of_nonempty(s)
            let f_seq: Nat -> T satisfy {
                forall(x: T) {
                    s.contains(x) implies exists(n: Nat) { f_seq(n) = x }
                }
            }
            let g: Nat -> U = function(n: Nat) { f(f_seq(n)) }
            is_countable[U](set_image(s, f)) = (set_image(s, f) = Set[U].empty_set or exists(h: Nat -> U) {
                forall(y: U) {
                    set_image(s, f).contains(y) implies exists(n: Nat) { h(n) = y }
                }
            })
            forall(y: U) {
                if set_image(s, f).contains(y) {
                    set_image_contains_witness(s, f, y)
                    let x: T satisfy {
                        s.contains(x) and y = f(x)
                    }
                    let n: Nat satisfy { f_seq(n) = x }
                    g(n) = f(f_seq(n))
                    g(n) = f(x)
                    g(n) = y
                    exists(m: Nat) { g(m) = y }
                }
            }
            forall(y: U) {
                set_image(s, f).contains(y) implies exists(m: Nat) { g(m) = y }
            }
            exists_intro(function(h: Nat -> U) {
                forall(y: U) {
                    set_image(s, f).contains(y) implies exists(n: Nat) { h(n) = y }
                }
            }, g)
            is_countable[U](set_image(s, f))
        }
    }
}

/// Countability of a type transfers along a bijection of types.
///
/// If the whole type `T` is countable and `f` is a bijection from `T` to `U`,
/// then the whole type `U` is countable: the image of the universal set under
/// the surjective map `f` is the universal set of `U`.
theorem countable_type_transfer_along_bijection[T, U](f: T -> U) {
    is_countable[T](Set[T].universal_set) and is_bijection_fn(f) implies
        is_countable[U](Set[U].universal_set)
} by {
    if is_countable[T](Set[T].universal_set) and is_bijection_fn(f) {
        countable_transfer_along_bijection(Set[T].universal_set, f)
        is_countable[U](set_image(Set[T].universal_set, f))
        bijection_fn_is_surjective(f)
        set_image_universal_of_surjective(f)
        set_image(Set[T].universal_set, f) = Set[U].universal_set
        is_countable[U](Set[U].universal_set)
    }
}

/// A set whose elements are all hit by a sequence is countable.
///
/// This is the introduction rule for countability: to show a set is countable
/// it suffices to exhibit a sequence that reaches every element of the set.
theorem sequence_covered_set_is_countable[T](s: Set[T], f: Nat -> T) {
    (forall(x: T) {
        s.contains(x) implies exists(n: Nat) { f(n) = x }
    }) implies is_countable[T](s)
} by {
    if forall(x: T) {
        s.contains(x) implies exists(n: Nat) { f(n) = x }
    } {
        nat_universal_is_countable
        image_of_countable_is_countable(Set[Nat].universal_set, f)
        is_countable[T](set_image(Set[Nat].universal_set, f))
        forall(x: T) {
            if s.contains(x) {
                let n: Nat satisfy { f(n) = x }
                universal_set_contains_eq[Nat](n)
                maps_into_set_image(Set[Nat].universal_set, f, n)
                set_image(Set[Nat].universal_set, f).contains(f(n))
                set_image(Set[Nat].universal_set, f).contains(x)
            }
        }
        s.subset(set_image(Set[Nat].universal_set, f))
        subset_of_countable_is_countable(s, set_image(Set[Nat].universal_set, f))
        is_countable[T](s)
    }
}

/// The enumeration of pairs of elements from two sequences, along the diagonal
/// enumeration of `Nat × Nat`.
define pair_enum[T, U](f: Nat -> T, g: Nat -> U, n: Nat) -> Pair[T, U] {
    Pair.new(f(nat_pair_zigzag(n).first), g(nat_pair_zigzag(n).second))
}

/// The Cartesian product of two countable sets is countable.
///
/// This is |A × B| = |A| · |B| on the level of countability: pairing the two
/// enumerating sequences with the diagonal enumeration of `Nat × Nat` covers
/// every pair of elements, so the product set is a subset of a countable image.
theorem pair_set_of_countable_is_countable[T, U](s: Set[T], t: Set[U]) {
    is_countable[T](s) and is_countable[U](t) implies is_countable[Pair[T, U]](pair_set(s, t))
} by {
    if is_countable[T](s) and is_countable[U](t) {
        if s = Set[T].empty_set {
            forall(p: Pair[T, U]) {
                pair_set_contains_eq(Set[T].empty_set, t, p)
                empty_set_contains_eq[T](p.first)
                empty_set_contains_eq[Pair[T, U]](p)
                pair_set(Set[T].empty_set, t).contains(p) = Set[Pair[T, U]].empty_set.contains(p)
            }
            set_ext(pair_set(Set[T].empty_set, t), Set[Pair[T, U]].empty_set)
            empty_set_is_countable[Pair[T, U]]
            is_countable[Pair[T, U]](pair_set(s, t))
        } else {
            if t = Set[U].empty_set {
                forall(p: Pair[T, U]) {
                    pair_set_contains_eq(s, Set[U].empty_set, p)
                    empty_set_contains_eq[U](p.second)
                    empty_set_contains_eq[Pair[T, U]](p)
                    pair_set(s, Set[U].empty_set).contains(p) = Set[Pair[T, U]].empty_set.contains(p)
                }
                set_ext(pair_set(s, Set[U].empty_set), Set[Pair[T, U]].empty_set)
                empty_set_is_countable[Pair[T, U]]
                is_countable[Pair[T, U]](pair_set(s, t))
            } else {
                countable_has_sequence_of_nonempty(s)
                countable_has_sequence_of_nonempty(t)
                let f: Nat -> T satisfy {
                    forall(x: T) {
                        s.contains(x) implies exists(n: Nat) { f(n) = x }
                    }
                }
                let g: Nat -> U satisfy {
                    forall(y: U) {
                        t.contains(y) implies exists(n: Nat) { g(n) = y }
                    }
                }
                nat_universal_is_countable
                image_of_countable_is_countable(Set[Nat].universal_set, pair_enum(f, g))
                is_countable[Pair[T, U]](set_image(Set[Nat].universal_set, pair_enum(f, g)))
                forall(p: Pair[T, U]) {
                    if pair_set(s, t).contains(p) {
                        pair_set_contains_imp_left(s, t, p)
                        pair_set_contains_imp_right(s, t, p)
                        let i: Nat satisfy { f(i) = p.first }
                        let j: Nat satisfy { g(j) = p.second }
                        nat_pair_zigzag_surjective
                        surjective_fn_has_preimage(nat_pair_zigzag, Pair.new(i, j))
                        let k: Nat satisfy { nat_pair_zigzag(k) = Pair.new(i, j) }
                        nat_pair_zigzag(k).first = Pair.new(i, j).first
                        pair_new_first(i, j)
                        Pair.new(i, j).first = i
                        nat_pair_zigzag(k).first = i
                        nat_pair_zigzag(k).second = Pair.new(i, j).second
                        pair_new_second(i, j)
                        Pair.new(i, j).second = j
                        nat_pair_zigzag(k).second = j
                        pair_enum(f, g, k) = Pair.new(f(nat_pair_zigzag(k).first),
                            g(nat_pair_zigzag(k).second))
                        pair_enum(f, g, k) = Pair.new(f(i), g(j))
                        f(i) = p.first
                        g(j) = p.second
                        pair_enum(f, g, k) = Pair.new(p.first, p.second)
                        pair_eta(p)
                        Pair.new(p.first, p.second) = p
                        pair_enum(f, g, k) = p
                        universal_set_contains_eq[Nat](k)
                        maps_into_set_image(Set[Nat].universal_set, pair_enum(f, g), k)
                        set_image(Set[Nat].universal_set, pair_enum(f, g)).contains(
                            pair_enum(f, g, k))
                        set_image(Set[Nat].universal_set, pair_enum(f, g)).contains(p)
                    }
                }
                pair_set(s, t).subset(set_image(Set[Nat].universal_set, pair_enum(f, g)))
                subset_of_countable_is_countable(pair_set(s, t),
                    set_image(Set[Nat].universal_set, pair_enum(f, g)))
                is_countable[Pair[T, U]](pair_set(s, t))
            }
        }
    }
}

/// The product of two countable types is countable.
theorem countable_type_product[T, U] {
    is_countable[T](Set[T].universal_set) and is_countable[U](Set[U].universal_set) implies
        is_countable[Pair[T, U]](Set[Pair[T, U]].universal_set)
} by {
    if is_countable[T](Set[T].universal_set) and is_countable[U](Set[U].universal_set) {
        pair_set_of_countable_is_countable(Set[T].universal_set, Set[U].universal_set)
        is_countable[Pair[T, U]](pair_set(Set[T].universal_set, Set[U].universal_set))
        forall(p: Pair[T, U]) {
            pair_set_contains_eq(Set[T].universal_set, Set[U].universal_set, p)
            universal_set_contains_eq[T](p.first)
            universal_set_contains_eq[U](p.second)
            universal_set_contains_eq[Pair[T, U]](p)
            pair_set(Set[T].universal_set, Set[U].universal_set).contains(p) =
                Set[Pair[T, U]].universal_set.contains(p)
        }
        set_ext(pair_set(Set[T].universal_set, Set[U].universal_set),
            Set[Pair[T, U]].universal_set)
        is_countable[Pair[T, U]](Set[Pair[T, U]].universal_set)
    }
}

/// The sequence of left injections of a sequence of elements.
define sum_left_enum[T, U](f: Nat -> T, n: Nat) -> Sum[T, U] {
    Sum.inl[T, U](f(n))
}

/// The sequence of right injections of a sequence of elements.
define sum_right_enum[T, U](g: Nat -> U, n: Nat) -> Sum[T, U] {
    Sum.inr[T, U](g(n))
}

/// The disjoint union of two countable sets is countable.
///
/// This is |A ⊔ B| = |A| + |B| on the level of countability: interleaving the
/// left and right injections of the two enumerating sequences covers every
/// element of the disjoint union.
theorem sum_set_of_countable_is_countable[T, U](s: Set[T], t: Set[U]) {
    is_countable[T](s) and is_countable[U](t) implies is_countable[Sum[T, U]](sum_set(s, t))
} by {
    if is_countable[T](s) and is_countable[U](t) {
        if s = Set[T].empty_set {
            if t = Set[U].empty_set {
                forall(z: Sum[T, U]) {
                    if sum_set(Set[T].empty_set, Set[U].empty_set).contains(z) {
                        match z {
                            Sum.inl(x) {
                                sum_set_contains_inl(Set[T].empty_set, Set[U].empty_set, x)
                                empty_set_contains_eq[T](x)
                                false
                            }
                            Sum.inr(y) {
                                sum_set_contains_inr(Set[T].empty_set, Set[U].empty_set, y)
                                empty_set_contains_eq[U](y)
                                false
                            }
                        }
                    }
                    empty_set_contains_eq[Sum[T, U]](z)
                    sum_set(Set[T].empty_set, Set[U].empty_set).contains(z) =
                        Set[Sum[T, U]].empty_set.contains(z)
                }
                set_ext(sum_set(Set[T].empty_set, Set[U].empty_set), Set[Sum[T, U]].empty_set)
                empty_set_is_countable[Sum[T, U]]
                is_countable[Sum[T, U]](sum_set(s, t))
            } else {
                countable_has_sequence_of_nonempty(t)
                let g: Nat -> U satisfy {
                    forall(y: U) {
                        t.contains(y) implies exists(n: Nat) { g(n) = y }
                    }
                }
                nat_universal_is_countable
                image_of_countable_is_countable(Set[Nat].universal_set, sum_right_enum[T, U](g))
                is_countable[Sum[T, U]](set_image(Set[Nat].universal_set,
                    sum_right_enum[T, U](g)))
                forall(z: Sum[T, U]) {
                    if sum_set(s, t).contains(z) {
                        match z {
                            Sum.inl(x) {
                                sum_set_contains_inl(s, t, x)
                                empty_set_contains_eq[T](x)
                                false
                            }
                            Sum.inr(y) {
                                sum_set_contains_inr(s, t, y)
                                t.contains(y)
                                exists(n: Nat) { g(n) = y }
                                let n: Nat satisfy { g(n) = y }
                                sum_right_enum[T, U](g, n) = Sum.inr[T, U](g(n))
                                sum_right_enum[T, U](g, n) = Sum.inr[T, U](y)
                                sum_right_enum[T, U](g, n) = z
                                universal_set_contains_eq[Nat](n)
                                maps_into_set_image(Set[Nat].universal_set,
                                    sum_right_enum[T, U](g), n)
                                set_image(Set[Nat].universal_set,
                                    sum_right_enum[T, U](g)).contains(sum_right_enum[T, U](g, n))
                                set_image(Set[Nat].universal_set,
                                    sum_right_enum[T, U](g)).contains(z)
                            }
                        }
                    }
                }
                sum_set(s, t).subset(set_image(Set[Nat].universal_set, sum_right_enum[T, U](g)))
                subset_of_countable_is_countable(sum_set(s, t),
                    set_image(Set[Nat].universal_set, sum_right_enum[T, U](g)))
                is_countable[Sum[T, U]](sum_set(s, t))
            }
        } else {
            if t = Set[U].empty_set {
                countable_has_sequence_of_nonempty(s)
                let f: Nat -> T satisfy {
                    forall(x: T) {
                        s.contains(x) implies exists(n: Nat) { f(n) = x }
                    }
                }
                nat_universal_is_countable
                image_of_countable_is_countable(Set[Nat].universal_set, sum_left_enum[T, U](f))
                is_countable[Sum[T, U]](set_image(Set[Nat].universal_set,
                    sum_left_enum[T, U](f)))
                forall(z: Sum[T, U]) {
                    if sum_set(s, t).contains(z) {
                        match z {
                            Sum.inl(x) {
                                sum_set_contains_inl(s, t, x)
                                s.contains(x)
                                exists(n: Nat) { f(n) = x }
                                let n: Nat satisfy { f(n) = x }
                                sum_left_enum[T, U](f, n) = Sum.inl[T, U](f(n))
                                sum_left_enum[T, U](f, n) = Sum.inl[T, U](x)
                                sum_left_enum[T, U](f, n) = z
                                universal_set_contains_eq[Nat](n)
                                maps_into_set_image(Set[Nat].universal_set,
                                    sum_left_enum[T, U](f), n)
                                set_image(Set[Nat].universal_set,
                                    sum_left_enum[T, U](f)).contains(sum_left_enum[T, U](f, n))
                                set_image(Set[Nat].universal_set,
                                    sum_left_enum[T, U](f)).contains(z)
                            }
                            Sum.inr(y) {
                                sum_set_contains_inr(s, t, y)
                                empty_set_contains_eq[U](y)
                                false
                            }
                        }
                    }
                }
                sum_set(s, t).subset(set_image(Set[Nat].universal_set, sum_left_enum[T, U](f)))
                subset_of_countable_is_countable(sum_set(s, t),
                    set_image(Set[Nat].universal_set, sum_left_enum[T, U](f)))
                is_countable[Sum[T, U]](sum_set(s, t))
            } else {
                countable_has_sequence_of_nonempty(s)
                countable_has_sequence_of_nonempty(t)
                let f: Nat -> T satisfy {
                    forall(x: T) {
                        s.contains(x) implies exists(n: Nat) { f(n) = x }
                    }
                }
                let g: Nat -> U satisfy {
                    forall(y: U) {
                        t.contains(y) implies exists(n: Nat) { g(n) = y }
                    }
                }
                let h: Nat -> Sum[T, U] = interleave_sequences(sum_left_enum[T, U](f),
                    sum_right_enum[T, U](g))
                nat_universal_is_countable
                image_of_countable_is_countable(Set[Nat].universal_set, h)
                is_countable[Sum[T, U]](set_image(Set[Nat].universal_set, h))
                forall(z: Sum[T, U]) {
                    if sum_set(s, t).contains(z) {
                        match z {
                            Sum.inl(x) {
                                sum_set_contains_inl(s, t, x)
                                let n: Nat satisfy { f(n) = x }
                                interleave_sequences_left_index(sum_left_enum[T, U](f),
                                    sum_right_enum[T, U](g), n)
                                h(countable_even_index(n)) = sum_left_enum[T, U](f, n)
                                sum_left_enum[T, U](f, n) = Sum.inl[T, U](f(n))
                                h(countable_even_index(n)) = Sum.inl[T, U](f(n))
                                h(countable_even_index(n)) = Sum.inl[T, U](x)
                                h(countable_even_index(n)) = z
                                universal_set_contains_eq[Nat](countable_even_index(n))
                                maps_into_set_image(Set[Nat].universal_set, h,
                                    countable_even_index(n))
                                set_image(Set[Nat].universal_set, h).contains(
                                    h(countable_even_index(n)))
                                set_image(Set[Nat].universal_set, h).contains(z)
                            }
                            Sum.inr(y) {
                                sum_set_contains_inr(s, t, y)
                                let n: Nat satisfy { g(n) = y }
                                interleave_sequences_right_index(sum_left_enum[T, U](f),
                                    sum_right_enum[T, U](g), n)
                                h(countable_odd_index(n)) = sum_right_enum[T, U](g, n)
                                sum_right_enum[T, U](g, n) = Sum.inr[T, U](g(n))
                                h(countable_odd_index(n)) = Sum.inr[T, U](g(n))
                                h(countable_odd_index(n)) = Sum.inr[T, U](y)
                                h(countable_odd_index(n)) = z
                                universal_set_contains_eq[Nat](countable_odd_index(n))
                                maps_into_set_image(Set[Nat].universal_set, h,
                                    countable_odd_index(n))
                                set_image(Set[Nat].universal_set, h).contains(
                                    h(countable_odd_index(n)))
                                set_image(Set[Nat].universal_set, h).contains(z)
                            }
                        }
                    }
                }
                sum_set(s, t).subset(set_image(Set[Nat].universal_set, h))
                subset_of_countable_is_countable(sum_set(s, t),
                    set_image(Set[Nat].universal_set, h))
                is_countable[Sum[T, U]](sum_set(s, t))
            }
        }
    }
}

/// The disjoint union of two countable types is countable.
theorem countable_type_sum[T, U] {
    is_countable[T](Set[T].universal_set) and is_countable[U](Set[U].universal_set) implies
        is_countable[Sum[T, U]](Set[Sum[T, U]].universal_set)
} by {
    if is_countable[T](Set[T].universal_set) and is_countable[U](Set[U].universal_set) {
        sum_set_of_countable_is_countable(Set[T].universal_set, Set[U].universal_set)
        is_countable[Sum[T, U]](sum_set(Set[T].universal_set, Set[U].universal_set))
        forall(z: Sum[T, U]) {
            match z {
                Sum.inl(x) {
                    sum_set_contains_inl(Set[T].universal_set, Set[U].universal_set, x)
                    universal_set_contains_eq[T](x)
                    universal_set_contains_eq[Sum[T, U]](z)
                    sum_set(Set[T].universal_set, Set[U].universal_set).contains(z) =
                        Set[Sum[T, U]].universal_set.contains(z)
                }
                Sum.inr(y) {
                    sum_set_contains_inr(Set[T].universal_set, Set[U].universal_set, y)
                    universal_set_contains_eq[U](y)
                    universal_set_contains_eq[Sum[T, U]](z)
                    sum_set(Set[T].universal_set, Set[U].universal_set).contains(z) =
                        Set[Sum[T, U]].universal_set.contains(z)
                }
            }
        }
        set_ext(sum_set(Set[T].universal_set, Set[U].universal_set),
            Set[Sum[T, U]].universal_set)
        is_countable[Sum[T, U]](Set[Sum[T, U]].universal_set)
    }
}

/// The finite list of length `n` whose contents come from the sequence `enum`,
/// with the `n` element indices and the shorter code packed into `code` by the
/// diagonal enumeration.
define list_code_enum[T](enum: Nat -> T, n: Nat, code: Nat) -> List[T] {
    match n {
        Nat.zero {
            List.nil[T]
        }
        Nat.suc(pred) {
            List.cons(enum(nat_pair_zigzag(code).first),
                list_code_enum(enum, pred, nat_pair_zigzag(code).second))
        }
    }
}

/// The `n`-th finite list in the enumeration over a surjective element
/// sequence: the length and the content code are unpacked from the diagonal
/// enumeration of `Nat × Nat`.
define list_enum[T](enum: Nat -> T, n: Nat) -> List[T] {
    list_code_enum(enum, nat_pair_zigzag(n).first, nat_pair_zigzag(n).second)
}

/// Every finite list of elements is produced by `list_code_enum` at its length.
///
/// The induction on the list: a cons list `cons(head, tail)` is produced at
/// length `tail.length + 1` from the code pairing an index of `head` with the
/// code of `tail`.
theorem list_code_enum_covers[T](enum: Nat -> T, l: List[T]) {
    (forall(x: T) { exists(i: Nat) { enum(i) = x } }) implies
        exists(code: Nat) { list_code_enum(enum, l.length, code) = l }
} by {
    if forall(x: T) { exists(i: Nat) { enum(i) = x } } {
        define p(xs: List[T]) -> Bool {
            exists(code: Nat) { list_code_enum(enum, xs.length, code) = xs }
        }
        list_code_enum(enum, List.nil[T].length, Nat.0) = List.nil[T]
        p(List.nil[T])
        forall(head: T, tail: List[T]) {
            if p(tail) {
                let m: Nat satisfy { list_code_enum(enum, tail.length, m) = tail }
                exists(i: Nat) { enum(i) = head }
                let i: Nat satisfy { enum(i) = head }
                nat_pair_zigzag_surjective
                surjective_fn_has_preimage(nat_pair_zigzag, Pair.new(i, m))
                let k: Nat satisfy { nat_pair_zigzag(k) = Pair.new(i, m) }
                nat_pair_zigzag(k).first = Pair.new(i, m).first
                pair_new_first(i, m)
                Pair.new(i, m).first = i
                nat_pair_zigzag(k).first = i
                nat_pair_zigzag(k).second = Pair.new(i, m).second
                pair_new_second(i, m)
                Pair.new(i, m).second = m
                nat_pair_zigzag(k).second = m
                list_code_enum(enum, List.cons(head, tail).length, k) =
                    list_code_enum(enum, tail.length.suc, k)
                list_code_enum(enum, tail.length.suc, k) =
                    List.cons(enum(nat_pair_zigzag(k).first),
                        list_code_enum(enum, tail.length, nat_pair_zigzag(k).second))
                enum(i) = head
                list_code_enum(enum, tail.length, m) = tail
                list_code_enum(enum, List.cons(head, tail).length, k) =
                    List.cons(head, tail)
                exists(code: Nat) {
                    list_code_enum(enum, List.cons(head, tail).length, code) =
                        List.cons(head, tail)
                }
                p(List.cons(head, tail))
            }
        }
        List.induction(function(xs: List[T]) { p(xs) })
        p(l)
        p(l) = exists(code: Nat) { list_code_enum(enum, l.length, code) = l }
        exists(code: Nat) { list_code_enum(enum, l.length, code) = l }
    }
}

/// Every finite list of elements is produced by `list_enum` at some index.
theorem list_enum_covers[T](enum: Nat -> T, l: List[T]) {
    (forall(x: T) { exists(i: Nat) { enum(i) = x } }) implies
        exists(n: Nat) { list_enum(enum, n) = l }
} by {
    if forall(x: T) { exists(i: Nat) { enum(i) = x } } {
        list_code_enum_covers(enum, l)
        let m: Nat satisfy { list_code_enum(enum, l.length, m) = l }
        nat_pair_zigzag_surjective
        surjective_fn_has_preimage(nat_pair_zigzag, Pair.new(l.length, m))
        let k: Nat satisfy { nat_pair_zigzag(k) = Pair.new(l.length, m) }
        nat_pair_zigzag(k).first = Pair.new(l.length, m).first
        pair_new_first(l.length, m)
        Pair.new(l.length, m).first = l.length
        nat_pair_zigzag(k).first = l.length
        nat_pair_zigzag(k).second = Pair.new(l.length, m).second
        pair_new_second(l.length, m)
        Pair.new(l.length, m).second = m
        nat_pair_zigzag(k).second = m
        list_enum(enum, k) = list_code_enum(enum, nat_pair_zigzag(k).first,
            nat_pair_zigzag(k).second)
        list_enum(enum, k) = list_code_enum(enum, l.length, m)
        list_enum(enum, k) = l
        exists(n: Nat) { list_enum(enum, n) = l }
    }
}

/// If the element type has no elements, every finite list is the empty list.
theorem lists_of_empty_type_are_nil[T] {
    Set[T].universal_set = Set[T].empty_set implies
        forall(l: List[T]) { l = List.nil[T] }
} by {
    if Set[T].universal_set = Set[T].empty_set {
        forall(l: List[T]) {
            match l {
                List.nil {
                    l = List.nil[T]
                }
                List.cons(head, tail) {
                    universal_set_contains_eq[T](head)
                    set_eq_contains_at(Set[T].universal_set, Set[T].empty_set, head)
                    empty_set_contains_eq[T](head)
                    false
                }
            }
        }
    }
}

/// The constant sequence of the empty list.
define nil_sequence[T](n: Nat) -> List[T] {
    List.nil[T]
}

/// The set of finite sequences over a countable set is countable.
///
/// Each finite list is determined by its length and the indices of its
/// elements in the enumerating sequence, so the lists form a subset of the
/// countable image of the diagonal enumeration of `Nat × Nat`.
theorem finite_sequences_of_countable_are_countable[T] {
    is_countable[T](Set[T].universal_set) implies
        is_countable[List[T]](Set[List[T]].universal_set)
} by {
    if is_countable[T](Set[T].universal_set) {
        if Set[T].universal_set = Set[T].empty_set {
            lists_of_empty_type_are_nil[T]
            forall(l: List[T]) { l = List.nil[T] }
            nat_universal_is_countable
            image_of_countable_is_countable(Set[Nat].universal_set, nil_sequence[T])
            is_countable[List[T]](set_image(Set[Nat].universal_set, nil_sequence[T]))
            forall(l: List[T]) {
                if Set[List[T]].universal_set.contains(l) {
                    l = List.nil[T]
                    nil_sequence[T](Nat.0) = List.nil[T]
                    nil_sequence[T](Nat.0) = l
                    universal_set_contains_eq[Nat](Nat.0)
                    maps_into_set_image(Set[Nat].universal_set, nil_sequence[T], Nat.0)
                    set_image(Set[Nat].universal_set, nil_sequence[T]).contains(
                        nil_sequence[T](Nat.0))
                    set_image(Set[Nat].universal_set, nil_sequence[T]).contains(l)
                }
            }
            Set[List[T]].universal_set.subset(set_image(Set[Nat].universal_set,
                nil_sequence[T]))
            subset_of_countable_is_countable(Set[List[T]].universal_set,
                set_image(Set[Nat].universal_set, nil_sequence[T]))
            is_countable[List[T]](Set[List[T]].universal_set)
        } else {
            countable_has_sequence_of_nonempty(Set[T].universal_set)
            let enum: Nat -> T satisfy {
                forall(x: T) {
                    Set[T].universal_set.contains(x) implies exists(n: Nat) { enum(n) = x }
                }
            }
            forall(x: T) {
                universal_set_contains_eq[T](x)
                exists(i: Nat) { enum(i) = x }
            }
            nat_universal_is_countable
            image_of_countable_is_countable(Set[Nat].universal_set, list_enum(enum))
            is_countable[List[T]](set_image(Set[Nat].universal_set, list_enum(enum)))
            forall(l: List[T]) {
                if Set[List[T]].universal_set.contains(l) {
                    list_enum_covers(enum, l)
                    let n: Nat satisfy { list_enum(enum, n) = l }
                    universal_set_contains_eq[Nat](n)
                    maps_into_set_image(Set[Nat].universal_set, list_enum(enum), n)
                    set_image(Set[Nat].universal_set, list_enum(enum)).contains(
                        list_enum(enum, n))
                    set_image(Set[Nat].universal_set, list_enum(enum)).contains(l)
                }
            }
            Set[List[T]].universal_set.subset(set_image(Set[Nat].universal_set,
                list_enum(enum)))
            subset_of_countable_is_countable(Set[List[T]].universal_set,
                set_image(Set[Nat].universal_set, list_enum(enum)))
            is_countable[List[T]](Set[List[T]].universal_set)
        }
    }
}
