/// Bridges for splitting list computations across `list_take` and `List.drop`.
///
/// The prefix/suffix API proves that `list_take items n + items.drop n`
/// reassembles `items`. This module packages the downstream consequences for
/// containment, length, counts, filtering, mapping, sums, products, and monoid
/// folds so finite-enumeration users can split a list at an index without
/// re-proving recursion facts.

from algebra.add_comm_monoid import AddCommMonoid
from algebra.comm_monoid import CommMonoid
from algebra.monoid.monoid import Monoid
from list import List, add_contains_right, add_length, count_append,
    filter_add, filter_contained_by_and, filter_contains_and, map, map_add,
    product, product_append, sum, sum_add
from data.list.list_fold_bridge import fold_right_monoid, fold_right_monoid_append
from data.list.list_prefix_suffix import list_take, list_take_append_drop,
    list_take_contains_imp_contains
from nat import Nat

/// Every element of the dropped suffix is an element of the original list.
theorem list_drop_contains_imp_contains[T](items: List[T], n: Nat, x: T) {
    items.drop(n).contains(x) implies items.contains(x)
} by {
    if items.drop(n).contains(x) {
        list_take_append_drop(items, n)
        list_take(items, n) + items.drop(n) = items
        add_contains_right(list_take(items, n), items.drop(n), x)
        (list_take(items, n) + items.drop(n)).contains(x)
        items.contains(x)
    }
}

/// The lengths of the taken prefix and dropped suffix add to the original length.
theorem list_take_drop_length_add[T](items: List[T], n: Nat) {
    list_take(items, n).length + items.drop(n).length = items.length
} by {
    list_take_append_drop(items, n)
    list_take(items, n) + items.drop(n) = items
    add_length(list_take(items, n), items.drop(n))
    list_take(items, n).length + items.drop(n).length =
        (list_take(items, n) + items.drop(n)).length
    (list_take(items, n) + items.drop(n)).length = items.length
    list_take(items, n).length + items.drop(n).length = items.length
}

/// The dropped suffix never has length greater than the original list.
theorem list_drop_length_at_most[T](items: List[T], n: Nat) {
    items.drop(n).length <= items.length
} by {
    list_take_drop_length_add(items, n)
    list_take(items, n).length + items.drop(n).length = items.length
    items.drop(n).length <= list_take(items, n).length + items.drop(n).length
    items.drop(n).length <= items.length
}

/// Counts split additively across the taken prefix and dropped suffix.
theorem list_take_drop_count_add[T](items: List[T], n: Nat, x: T) {
    list_take(items, n).count(x) + items.drop(n).count(x) = items.count(x)
} by {
    list_take_append_drop(items, n)
    list_take(items, n) + items.drop(n) = items
    count_append(list_take(items, n), items.drop(n), x)
    (list_take(items, n) + items.drop(n)).count(x) =
        list_take(items, n).count(x) + items.drop(n).count(x)
    (list_take(items, n) + items.drop(n)).count(x) = items.count(x)
    list_take(items, n).count(x) + items.drop(n).count(x) = items.count(x)
}

/// Filtering the taken prefix and dropped suffix reassembles the filtered list.
theorem list_take_drop_filter_add[T](items: List[T], n: Nat, pred: T -> Bool) {
    list_take(items, n).filter(pred) + items.drop(n).filter(pred) = items.filter(pred)
} by {
    list_take_append_drop(items, n)
    list_take(items, n) + items.drop(n) = items
    filter_add(list_take(items, n), items.drop(n), pred)
    (list_take(items, n) + items.drop(n)).filter(pred) =
        list_take(items, n).filter(pred) + items.drop(n).filter(pred)
    (list_take(items, n) + items.drop(n)).filter(pred) = items.filter(pred)
    list_take(items, n).filter(pred) + items.drop(n).filter(pred) = items.filter(pred)
}

/// A filtered taken-prefix member is a filtered member of the original list.
theorem list_take_filter_contains_imp_filter_contains[T](items: List[T], n: Nat,
    pred: T -> Bool, x: T) {
    list_take(items, n).filter(pred).contains(x) implies items.filter(pred).contains(x)
} by {
    if list_take(items, n).filter(pred).contains(x) {
        filter_contained_by_and(list_take(items, n), pred, x)
        list_take(items, n).contains(x)
        pred(x)
        list_take_contains_imp_contains(items, n, x)
        items.contains(x)
        filter_contains_and(items, pred, x)
        items.filter(pred).contains(x)
    }
}

/// A filtered dropped-suffix member is a filtered member of the original list.
theorem list_drop_filter_contains_imp_filter_contains[T](items: List[T], n: Nat,
    pred: T -> Bool, x: T) {
    items.drop(n).filter(pred).contains(x) implies items.filter(pred).contains(x)
} by {
    if items.drop(n).filter(pred).contains(x) {
        filter_contained_by_and(items.drop(n), pred, x)
        items.drop(n).contains(x)
        pred(x)
        list_drop_contains_imp_contains(items, n, x)
        items.contains(x)
        filter_contains_and(items, pred, x)
        items.filter(pred).contains(x)
    }
}

/// Mapping the taken prefix and dropped suffix reassembles the mapped list.
theorem list_take_drop_map_add[T, U](items: List[T], n: Nat, f: T -> U) {
    map[T, U](list_take(items, n), f) + map[T, U](items.drop(n), f) = map[T, U](items, f)
} by {
    list_take_append_drop(items, n)
    list_take(items, n) + items.drop(n) = items
    map_add(list_take(items, n), items.drop(n), f)
    map[T, U](list_take(items, n) + items.drop(n), f) =
        map[T, U](list_take(items, n), f) + map[T, U](items.drop(n), f)
    map[T, U](list_take(items, n) + items.drop(n), f) = map[T, U](items, f)
    map[T, U](list_take(items, n), f) + map[T, U](items.drop(n), f) = map[T, U](items, f)
}

/// Sums split additively across the taken prefix and dropped suffix.
theorem list_take_drop_sum_add[A: AddCommMonoid](items: List[A], n: Nat) {
    sum[A](list_take(items, n)) + sum[A](items.drop(n)) = sum[A](items)
} by {
    list_take_append_drop(items, n)
    list_take(items, n) + items.drop(n) = items
    sum_add(list_take(items, n), items.drop(n))
    sum[A](list_take(items, n) + items.drop(n)) =
        sum[A](list_take(items, n)) + sum[A](items.drop(n))
    sum[A](list_take(items, n) + items.drop(n)) = sum[A](items)
    sum[A](list_take(items, n)) + sum[A](items.drop(n)) = sum[A](items)
}

/// Mapped sums split additively across the taken prefix and dropped suffix.
theorem list_take_drop_sum_map_add[T, A: AddCommMonoid](items: List[T], n: Nat,
    f: T -> A) {
    sum[A](map[T, A](list_take(items, n), f)) + sum[A](map[T, A](items.drop(n), f)) =
        sum[A](map[T, A](items, f))
} by {
    list_take_drop_map_add(items, n, f)
    map[T, A](list_take(items, n), f) + map[T, A](items.drop(n), f) = map[T, A](items, f)
    sum_add(map[T, A](list_take(items, n), f), map[T, A](items.drop(n), f))
    sum[A](map[T, A](list_take(items, n), f) + map[T, A](items.drop(n), f)) =
        sum[A](map[T, A](list_take(items, n), f)) + sum[A](map[T, A](items.drop(n), f))
    sum[A](map[T, A](list_take(items, n), f) + map[T, A](items.drop(n), f)) =
        sum[A](map[T, A](items, f))
    sum[A](map[T, A](list_take(items, n), f)) + sum[A](map[T, A](items.drop(n), f)) =
        sum[A](map[T, A](items, f))
}

/// Products split multiplicatively across the taken prefix and dropped suffix.
theorem list_take_drop_product_mul[A: CommMonoid](items: List[A], n: Nat) {
    product[A](list_take(items, n)) * product[A](items.drop(n)) = product[A](items)
} by {
    list_take_append_drop(items, n)
    list_take(items, n) + items.drop(n) = items
    product_append(list_take(items, n), items.drop(n))
    product[A](list_take(items, n) + items.drop(n)) =
        product[A](list_take(items, n)) * product[A](items.drop(n))
    product[A](list_take(items, n) + items.drop(n)) = product[A](items)
    product[A](list_take(items, n)) * product[A](items.drop(n)) = product[A](items)
}

/// Monoid right folds split multiplicatively across the taken prefix and dropped suffix.
theorem list_take_drop_fold_right_monoid_mul[T, A: Monoid](items: List[T], n: Nat,
    f: T -> A) {
    fold_right_monoid[T, A](list_take(items, n), f) *
        fold_right_monoid[T, A](items.drop(n), f) = fold_right_monoid[T, A](items, f)
} by {
    list_take_append_drop(items, n)
    list_take(items, n) + items.drop(n) = items
    fold_right_monoid_append(list_take(items, n), items.drop(n), f)
    fold_right_monoid[T, A](list_take(items, n) + items.drop(n), f) =
        fold_right_monoid[T, A](list_take(items, n), f) *
        fold_right_monoid[T, A](items.drop(n), f)
    fold_right_monoid[T, A](list_take(items, n) + items.drop(n), f) =
        fold_right_monoid[T, A](items, f)
    fold_right_monoid[T, A](list_take(items, n), f) *
        fold_right_monoid[T, A](items.drop(n), f) = fold_right_monoid[T, A](items, f)
}
