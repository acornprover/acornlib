from list import List, filter_preserves_unique, is_permutation,
    list_contains_implies_count_geq_one, list_not_contains_impl_count_zero,
    permutation_preserves_length, remove_elem_eq_filter
from nat import Nat, only_zero_lte_zero

numerals Nat

/// Count on a cons is the defining if-then-else on the head.
theorem count_cons_eq_if[T](head: T, tail: List[T], item: T) {
    List.cons(head, tail).count(item) = if head = item { 1 + tail.count(item) } else { tail.count(item) }
}

/// Counting an item equal to the head of a cons adds one to the tail count.
theorem count_cons_of_eq[T](head: T, tail: List[T], item: T) {
    head = item implies List.cons(head, tail).count(item) = 1 + tail.count(item)
} by {
    if head = item {
        count_cons_eq_if(head, tail, item)
        List.cons(head, tail).count(item) = if head = item { 1 + tail.count(item) } else { tail.count(item) }
        List.cons(head, tail).count(item) = 1 + tail.count(item)
    }
}

/// Counting an item distinct from the head of a cons is the same as counting it in the tail.
theorem count_cons_other[T](head: T, tail: List[T], item: T) {
    head != item implies List.cons(head, tail).count(item) = tail.count(item)
} by {
    if head != item {
        count_cons_eq_if(head, tail, item)
        List.cons(head, tail).count(item) = if head = item { 1 + tail.count(item) } else { tail.count(item) }
        List.cons(head, tail).count(item) = tail.count(item)
    }
}

/// Filtering a cons whose head satisfies the predicate keeps the head.
theorem filter_cons_of_true[T](head: T, tail: List[T], pred: T -> Bool) {
    pred(head) implies List.cons(head, tail).filter(pred) = List.cons(head, tail.filter(pred))
}

/// Filtering a cons whose head does not satisfy the predicate removes the head.
theorem filter_cons_of_false[T](head: T, tail: List[T], pred: T -> Bool) {
    not pred(head) implies List.cons(head, tail).filter(pred) = tail.filter(pred)
}

/// Filtering by a predicate that is true at `item` preserves the count of `item`.
theorem filter_count_of_true[T](items: List[T], pred: T -> Bool, item: T) {
    pred(item) implies items.filter(pred).count(item) = items.count(item)
} by {
    define p(xs: List[T]) -> Bool {
        pred(item) implies xs.filter(pred).count(item) = xs.count(item)
    }

    p(List.nil[T])

    forall(head: T, tail: List[T]) {
        if p(tail) {
            if pred(item) {
                tail.filter(pred).count(item) = tail.count(item)

                if pred(head) {
                    List.cons(head, tail).filter(pred) = List.cons(head, tail.filter(pred))
                    if head = item {
                        count_cons_of_eq(head, tail.filter(pred), item)
                        List.cons(head, tail.filter(pred)).count(item) = 1 + tail.filter(pred).count(item)
                        List.cons(head, tail).filter(pred).count(item) =
                            1 + tail.filter(pred).count(item)
                        count_cons_of_eq(head, tail, item)
                        List.cons(head, tail).count(item) = 1 + tail.count(item)
                        1 + tail.filter(pred).count(item) = 1 + tail.count(item)
                        List.cons(head, tail).filter(pred).count(item) =
                            List.cons(head, tail).count(item)
                    } else {
                        count_cons_other(head, tail.filter(pred), item)
                        List.cons(head, tail.filter(pred)).count(item) = tail.filter(pred).count(item)
                        List.cons(head, tail).filter(pred).count(item) = tail.filter(pred).count(item)
                        count_cons_other(head, tail, item)
                        List.cons(head, tail).count(item) = tail.count(item)
                        List.cons(head, tail).filter(pred).count(item) =
                            List.cons(head, tail).count(item)
                    }
                } else {
                    if head = item {
                        pred(head)
                        false
                    }
                    head != item
                    List.cons(head, tail).filter(pred) = tail.filter(pred)
                    List.cons(head, tail).count(item) = tail.count(item)
                    List.cons(head, tail).filter(pred).count(item) =
                        List.cons(head, tail).count(item)
                }
            }
            p(List.cons(head, tail))
        }
    }

    List.induction(function(xs: List[T]) { p(xs) })
    forall(xs: List[T]) {
        p(xs)
    }
    p(items)
}

/// Filtering by a predicate that is false at `item` removes every occurrence of `item`.
theorem filter_count_of_false[T](items: List[T], pred: T -> Bool, item: T) {
    not pred(item) implies items.filter(pred).count(item) = Nat.0
} by {
    define p(xs: List[T]) -> Bool {
        not pred(item) implies xs.filter(pred).count(item) = Nat.0
    }

    p(List.nil[T])

    forall(head: T, tail: List[T]) {
        if p(tail) {
            if not pred(item) {
                tail.filter(pred).count(item) = Nat.0

                if pred(head) {
                    if head = item {
                        pred(item)
                        false
                    }
                    head != item
                    List.cons(head, tail).filter(pred) = List.cons(head, tail.filter(pred))
                    count_cons_other(head, tail.filter(pred), item)
                    List.cons(head, tail.filter(pred)).count(item) = tail.filter(pred).count(item)
                    List.cons(head, tail).filter(pred).count(item) = tail.filter(pred).count(item)
                    List.cons(head, tail).filter(pred).count(item) = Nat.0
                } else {
                    List.cons(head, tail).filter(pred) = tail.filter(pred)
                    List.cons(head, tail).filter(pred).count(item) = Nat.0
                }
            }
            p(List.cons(head, tail))
        }
    }

    List.induction(function(xs: List[T]) { p(xs) })
    forall(xs: List[T]) {
        p(xs)
    }
    p(items)
}

/// Filtering has a pointwise count formula: a kept element retains its count,
/// and a rejected element has count zero in the filtered list.
theorem filter_count_eq_if[T](items: List[T], pred: T -> Bool, item: T) {
    items.filter(pred).count(item) = if pred(item) { items.count(item) } else { Nat.0 }
} by {
    if pred(item) {
        filter_count_of_true(items, pred, item)
        items.filter(pred).count(item) = items.count(item)
        items.filter(pred).count(item) = if pred(item) { items.count(item) } else { Nat.0 }
    } else {
        filter_count_of_false(items, pred, item)
        items.filter(pred).count(item) = Nat.0
        items.filter(pred).count(item) = if pred(item) { items.count(item) } else { Nat.0 }
    }
}

/// Filtering a list preserves permutation equivalence.
theorem filter_preserves_permutation[T](a: List[T], b: List[T], pred: T -> Bool) {
    is_permutation(a, b) implies is_permutation(a.filter(pred), b.filter(pred))
} by {
    if is_permutation(a, b) {
        forall(x: T) {
            filter_count_eq_if(a, pred, x)
            filter_count_eq_if(b, pred, x)
            if pred(x) {
                a.filter(pred).count(x) = a.count(x)
                b.filter(pred).count(x) = b.count(x)
                a.count(x) = b.count(x)
                a.filter(pred).count(x) = b.filter(pred).count(x)
            } else {
                a.filter(pred).count(x) = Nat.0
                b.filter(pred).count(x) = Nat.0
                a.filter(pred).count(x) = b.filter(pred).count(x)
            }
        }
        is_permutation(a.filter(pred), b.filter(pred))
    }
}

/// Permutation-equivalent lists have the same filtered length.
theorem permutation_filter_length[T](a: List[T], b: List[T], pred: T -> Bool) {
    is_permutation(a, b) implies a.filter(pred).length = b.filter(pred).length
} by {
    if is_permutation(a, b) {
        filter_preserves_permutation(a, b, pred)
        is_permutation(a.filter(pred), b.filter(pred))
        permutation_preserves_length(a.filter(pred), b.filter(pred))
        a.filter(pred).length = b.filter(pred).length
    }
}

/// Permutations preserve membership.
theorem permutation_preserves_contains[T](a: List[T], b: List[T], item: T) {
    is_permutation(a, b) implies a.contains(item) = b.contains(item)
} by {
    if is_permutation(a, b) {
        if a.contains(item) {
            list_contains_implies_count_geq_one(a, item)
            a.count(item) >= Nat.1
            a.count(item) = b.count(item)
            b.count(item) >= Nat.1
            if not b.contains(item) {
                list_not_contains_impl_count_zero(b, item)
                b.count(item) = Nat.0
                Nat.1 <= b.count(item)
                Nat.1 <= Nat.0
                only_zero_lte_zero(Nat.1)
                Nat.1 = Nat.0
                false
            }
            b.contains(item)
        }
        if b.contains(item) {
            list_contains_implies_count_geq_one(b, item)
            b.count(item) >= Nat.1
            a.count(item) = b.count(item)
            a.count(item) >= Nat.1
            if not a.contains(item) {
                list_not_contains_impl_count_zero(a, item)
                a.count(item) = Nat.0
                Nat.1 <= a.count(item)
                Nat.1 <= Nat.0
                only_zero_lte_zero(Nat.1)
                Nat.1 = Nat.0
                false
            }
            a.contains(item)
        }
        a.contains(item) implies b.contains(item)
        b.contains(item) implies a.contains(item)
        a.contains(item) = b.contains(item)
    }
}

/// Permutation-equivalent lists have filters with the same membership.
theorem permutation_filter_contains[T](a: List[T], b: List[T], pred: T -> Bool, item: T) {
    is_permutation(a, b) implies a.filter(pred).contains(item) = b.filter(pred).contains(item)
} by {
    if is_permutation(a, b) {
        filter_preserves_permutation(a, b, pred)
        permutation_preserves_contains(a.filter(pred), b.filter(pred), item)
    }
}

/// Filtering a unique permutation pair gives unique permutation-equivalent lists.
theorem unique_permutation_filter_preserves_unique[T](a: List[T], b: List[T], pred: T -> Bool) {
    a.is_unique and b.is_unique and is_permutation(a, b) implies
        a.filter(pred).is_unique and b.filter(pred).is_unique and
        is_permutation(a.filter(pred), b.filter(pred))
} by {
    if a.is_unique and b.is_unique and is_permutation(a, b) {
        filter_preserves_unique(a, pred)
        filter_preserves_unique(b, pred)
        filter_preserves_permutation(a, b, pred)
        a.filter(pred).is_unique
        b.filter(pred).is_unique
        is_permutation(a.filter(pred), b.filter(pred))
        a.filter(pred).is_unique and b.filter(pred).is_unique and
            is_permutation(a.filter(pred), b.filter(pred))
    }
}

/// Removing all copies of an element leaves that element with count zero.
theorem remove_elem_count_self_zero[T](items: List[T], elem: T) {
    items.remove_elem(elem).count(elem) = Nat.0
} by {
    remove_elem_eq_filter(items, elem)
    not function(x: T) { x != elem }(elem)
    filter_count_of_false(items, function(x: T) { x != elem }, elem)
    items.filter(function(x: T) { x != elem }).count(elem) = Nat.0
    items.remove_elem(elem) = items.filter(function(x: T) { x != elem })
    items.remove_elem(elem).count(elem) = Nat.0
}

/// Removing one element preserves the count of every distinct element.
theorem remove_elem_count_other[T](items: List[T], elem: T, item: T) {
    item != elem implies items.remove_elem(elem).count(item) = items.count(item)
} by {
    if item != elem {
        remove_elem_eq_filter(items, elem)
        function(x: T) { x != elem }(item)
        filter_count_of_true(items, function(x: T) { x != elem }, item)
        items.filter(function(x: T) { x != elem }).count(item) = items.count(item)
        items.remove_elem(elem) = items.filter(function(x: T) { x != elem })
        items.remove_elem(elem).count(item) = items.count(item)
    }
}

/// Removing all copies of the same element preserves permutation equivalence.
theorem remove_elem_preserves_permutation[T](a: List[T], b: List[T], elem: T) {
    is_permutation(a, b) implies is_permutation(a.remove_elem(elem), b.remove_elem(elem))
} by {
    if is_permutation(a, b) {
        filter_preserves_permutation(a, b, function(x: T) { x != elem })
        is_permutation(a.filter(function(x: T) { x != elem }), b.filter(function(x: T) { x != elem }))
        remove_elem_eq_filter(a, elem)
        remove_elem_eq_filter(b, elem)
        a.remove_elem(elem) = a.filter(function(x: T) { x != elem })
        b.remove_elem(elem) = b.filter(function(x: T) { x != elem })
        is_permutation(a.remove_elem(elem), b.remove_elem(elem))
    }
}

/// Permutation-equivalent lists have removal lists of the same length.
theorem remove_elem_permutation_length[T](a: List[T], b: List[T], elem: T) {
    is_permutation(a, b) implies a.remove_elem(elem).length = b.remove_elem(elem).length
} by {
    if is_permutation(a, b) {
        remove_elem_preserves_permutation(a, b, elem)
        is_permutation(a.remove_elem(elem), b.remove_elem(elem))
        permutation_preserves_length(a.remove_elem(elem), b.remove_elem(elem))
        a.remove_elem(elem).length = b.remove_elem(elem).length
    }
}
