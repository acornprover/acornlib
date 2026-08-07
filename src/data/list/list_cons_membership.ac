from nat import Nat
from list import List

numerals Nat

/// A prepended list contains an item exactly when the head matches or the tail contains it.
///
/// `contains` is defined by a `match` on nil and cons. The match reduces only inside a
/// case split on whether the head matches, which is why this needs to be stated once and
/// cited afterwards rather than left to proof search.
///
/// This belongs in the list package proper; it is here so the package interface is not
/// changed in the middle of unrelated work.
theorem cons_contains_eq[T](head: T, tail: List[T], item: T) {
    List.cons(head, tail).contains(item) = (head = item or tail.contains(item))
} by {
    if List.cons(head, tail).contains(item) {
        if head = item {
            head = item or tail.contains(item)
        } else {
            tail.contains(item)
            head = item or tail.contains(item)
        }
        head = item or tail.contains(item)
    }
    if head = item or tail.contains(item) {
        if head = item {
            List.cons(head, tail).contains(item)
        }
        if head != item {
            tail.contains(item)
            List.cons(head, tail).contains(item)
        }
        List.cons(head, tail).contains(item)
    }
}

/// A prepended list contains its own head.
theorem cons_contains_head[T](head: T, tail: List[T]) {
    List.cons(head, tail).contains(head)
} by {
    cons_contains_eq(head, tail, head)
    List.cons(head, tail).contains(head) = (head = head or tail.contains(head))
}

/// A prepended list contains everything its tail contains.
theorem cons_contains_of_tail_contains[T](head: T, tail: List[T], item: T) {
    tail.contains(item) implies List.cons(head, tail).contains(item)
} by {
    if tail.contains(item) {
        cons_contains_eq(head, tail, item)
        List.cons(head, tail).contains(item) = (head = item or tail.contains(item))
        List.cons(head, tail).contains(item)
    }
}

/// An item in a prepended list is the head or lies in the tail.
theorem cons_contains_cases[T](head: T, tail: List[T], item: T) {
    List.cons(head, tail).contains(item) implies head = item or tail.contains(item)
} by {
    cons_contains_eq(head, tail, item)
}

/// The empty list contains nothing.
theorem nil_not_contains[T](item: T) {
    not List.nil[T].contains(item)
}
