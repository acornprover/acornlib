from list import List, length_range, range_contains_iff_lt, range_is_unique,
    map, map_length
from nat import Nat

numerals Nat

/// Local range-length wrapper: the visible `Nat.range` has length `n`.
theorem list_range_length(n: Nat) {
    n.range.length = n
} by {
    length_range(n)
}

/// Local range membership wrapper: entries of `n.range` are exactly naturals below `n`.
theorem list_range_contains_iff_lt(n: Nat, x: Nat) {
    n.range.contains(x) = (x < n)
} by {
    range_contains_iff_lt(n, x)
}

/// Local range uniqueness wrapper: `Nat.range` has no duplicates.
theorem list_range_is_unique(n: Nat) {
    n.range.is_unique
} by {
    range_is_unique(n)
}

/// Add an offset to every element of a list of naturals.
define shift_range_list(a: Nat, xs: List[Nat]) -> List[Nat] {
    map(xs, function(x: Nat) { a + x })
}

/// Length of a natural-number list as a first-class map target.
define list_nat_length(xs: List[Nat]) -> Nat {
    xs.length
}

/// Shifting every element of a natural-number list preserves its length.
theorem shift_range_list_length(a: Nat, xs: List[Nat]) {
    shift_range_list(a, xs).length = xs.length
} by {
    shift_range_list(a, xs) = map(xs, function(x: Nat) { a + x })
    map_length(xs, function(x: Nat) { a + x })
}

/// Shifting each natural list in a list of lists preserves the mapped lengths.
theorem map_shift_range_list_length(a: Nat, blocks: List[List[Nat]]) {
    map(map(blocks, shift_range_list(a)), list_nat_length) = map(blocks, list_nat_length)
} by {
    define p(bs: List[List[Nat]]) -> Bool {
        map(map(bs, shift_range_list(a)), list_nat_length) = map(bs, list_nat_length)
    }

    p(List.nil[List[Nat]])

    forall(head: List[Nat], tail: List[List[Nat]]) {
        if p(tail) {
            map[List[Nat], List[Nat]](List.cons(head, tail), shift_range_list(a)) =
                List.cons(shift_range_list(a, head), map(tail, shift_range_list(a)))
            map[List[Nat], Nat](List.cons(shift_range_list(a, head), map(tail, shift_range_list(a))),
                list_nat_length) =
                List.cons(list_nat_length(shift_range_list(a, head)),
                    map(map(tail, shift_range_list(a)), list_nat_length))
            map[List[Nat], Nat](List.cons(head, tail), list_nat_length) =
                List.cons(list_nat_length(head), map(tail, list_nat_length))
            shift_range_list_length(a, head)
            list_nat_length(shift_range_list(a, head)) = list_nat_length(head)
            map(map(tail, shift_range_list(a)), list_nat_length) = map(tail, list_nat_length)
            p(List.cons(head, tail))
        }
    }
    p(blocks)
}

/// Consecutive range blocks with lengths prescribed by the input list.
define ranges(items: List[Nat]) -> List[List[Nat]] {
    match items {
        List.nil {
            List.nil[List[Nat]]
        }
        List.cons(a, tail) {
            List.cons(a.range, map(ranges(tail), shift_range_list(a)))
        }
    }
}

/// The lengths of `ranges items` recover `items`.
theorem ranges_length(items: List[Nat]) {
    map(ranges(items), list_nat_length) = items
} by {
    define p(xs: List[Nat]) -> Bool {
        map(ranges(xs), list_nat_length) = xs
    }

    p(List.nil[Nat])

    forall(a: Nat, tail: List[Nat]) {
        if p(tail) {
            ranges(List.cons(a, tail)) =
                List.cons(a.range, map(ranges(tail), shift_range_list(a)))
            map(ranges(List.cons(a, tail)), list_nat_length) =
                List.cons(list_nat_length(a.range),
                    map(map(ranges(tail), shift_range_list(a)), list_nat_length))
            length_range(a)
            list_nat_length(a.range) = a
            map_shift_range_list_length(a, ranges(tail))
            map(map(ranges(tail), shift_range_list(a)), list_nat_length) =
                map(ranges(tail), list_nat_length)
            map(ranges(tail), list_nat_length) = tail
            map(ranges(List.cons(a, tail)), list_nat_length) = List.cons(a, tail)
            p(List.cons(a, tail))
        }
    }
    p(items)
}
