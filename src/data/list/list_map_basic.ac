from list import List, map

/// The deprecated top-level `map` sends the empty list to the empty list.
theorem map_nil[T, U](f: T -> U) {
    map[T, U](List.nil[T], f) = List.nil[U]
}

/// The deprecated top-level `map` distributes over a cons constructor.
theorem map_cons[T, U](head: T, tail: List[T], f: T -> U) {
    map[T, U](List.cons(head, tail), f) = List.cons(f(head), map[T, U](tail, f))
}

/// The attribute-method `List.map` sends the empty list to the empty list.
theorem list_map_nil[T, U](f: T -> U) {
    List.nil[T].map(f) = List.nil[U]
}

/// The attribute-method `List.map` distributes over a cons constructor.
theorem list_map_cons[T, U](head: T, tail: List[T], f: T -> U) {
    List.cons(head, tail).map(f) = List.cons(f(head), tail.map(f))
}
