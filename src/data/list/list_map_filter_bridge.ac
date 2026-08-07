from list import List, filter_contained_by_and, filter_contains_and, map,
    map_contains, map_contains_of_contains

/// Mapping after filtering agrees with filtering after mapping when the predicates match pointwise.
theorem map_filter_of_pointwise[T, U](items: List[T], f: T -> U, p: T -> Bool, q: U -> Bool) {
    forall(x: T) { p(x) = q(f(x)) }
        implies map[T, U](items.filter(p), f) = map[T, U](items, f).filter(q)
} by {
    define r(xs: List[T]) -> Bool {
        map[T, U](xs.filter(p), f) = map[T, U](xs, f).filter(q)
    }

    if forall(x: T) { p(x) = q(f(x)) } {
        map[T, U](List.nil[T].filter(p), f) = List.nil[U]
        map[T, U](List.nil[T], f) = List.nil[U]
        map[T, U](List.nil[T], f).filter(q) = List.nil[U]
        r(List.nil[T])

        forall(head: T, tail: List[T]) {
            if r(tail) {
                p(head) = q(f(head))
                if p(head) {
                    q(f(head))
                    List.cons(head, tail).filter(p) = List.cons(head, tail.filter(p))
                    map[T, U](List.cons(head, tail).filter(p), f) =
                        map[T, U](List.cons(head, tail.filter(p)), f)
                    map[T, U](List.cons(head, tail.filter(p)), f) =
                        List.cons(f(head), map[T, U](tail.filter(p), f))
                    map[T, U](List.cons(head, tail), f) = List.cons(f(head), map[T, U](tail, f))
                    List.cons(f(head), map[T, U](tail, f)).filter(q) =
                        List.cons(f(head), map[T, U](tail, f).filter(q))
                    map[T, U](tail.filter(p), f) = map[T, U](tail, f).filter(q)
                    map[T, U](List.cons(head, tail).filter(p), f) =
                        map[T, U](List.cons(head, tail), f).filter(q)
                } else {
                    not q(f(head))
                    List.cons(head, tail).filter(p) = tail.filter(p)
                    map[T, U](List.cons(head, tail).filter(p), f) = map[T, U](tail.filter(p), f)
                    map[T, U](List.cons(head, tail), f) = List.cons(f(head), map[T, U](tail, f))
                    List.cons(f(head), map[T, U](tail, f)).filter(q) = map[T, U](tail, f).filter(q)
                    map[T, U](tail.filter(p), f) = map[T, U](tail, f).filter(q)
                    map[T, U](List.cons(head, tail).filter(p), f) =
                        map[T, U](List.cons(head, tail), f).filter(q)
                }
                r(List.cons(head, tail))
            }
        }

        List.induction(function(xs: List[T]) { r(xs) })
        forall(xs: List[T]) {
            r(xs)
        }
        r(items)
    }
}

/// Filtering a mapped list by a target predicate is the same as filtering first by its pullback.
theorem map_filter_self[T, U](items: List[T], f: T -> U, p: U -> Bool) {
    map[T, U](items.filter(function(x: T) { p(f(x)) }), f) = map[T, U](items, f).filter(p)
} by {
    forall(x: T) {
        function(y: T) { p(f(y)) }(x) = p(f(x))
    }
    map_filter_of_pointwise[T, U](items, f, function(x: T) { p(f(x)) }, p)
}

/// Membership in a mapped list is exactly having a preimage in the source list.
theorem map_contains_iff_exists_preimage[T, U](items: List[T], f: T -> U, item: U) {
    map[T, U](items, f).contains(item) = exists(x: T) {
        items.contains(x) and f(x) = item
    }
} by {
    if map[T, U](items, f).contains(item) {
        map_contains(items, f, item)
        exists(x: T) {
            items.contains(x) and f(x) = item
        }
    }
    if exists(x: T) { items.contains(x) and f(x) = item } {
        let x: T satisfy {
            items.contains(x) and f(x) = item
        }
        map_contains_of_contains(items, f, x)
        map[T, U](items, f).contains(f(x))
        map[T, U](items, f).contains(item)
    }
}

/// If every mapped source element satisfies a target predicate, filtering after mapping does nothing.
theorem map_filter_of_all_mapped[T, U](items: List[T], f: T -> U, pred: U -> Bool) {
    forall(x: T) { items.contains(x) implies pred(f(x)) } implies
        map[T, U](items, f).filter(pred) = map[T, U](items, f)
} by {
    define r(xs: List[T]) -> Bool {
        forall(x: T) { xs.contains(x) implies pred(f(x)) } implies
            map[T, U](xs, f).filter(pred) = map[T, U](xs, f)
    }

    if forall(x: T) { List.nil[T].contains(x) implies pred(f(x)) } {
        map[T, U](List.nil[T], f) = List.nil[U]
        map[T, U](List.nil[T], f).filter(pred) = List.nil[U]
        map[T, U](List.nil[T], f).filter(pred) = map[T, U](List.nil[T], f)
    }
    r(List.nil[T])

    forall(head: T, tail: List[T]) {
        if r(tail) {
            if forall(x: T) { List.cons(head, tail).contains(x) implies pred(f(x)) } {
                forall(x: T) {
                    if tail.contains(x) {
                        List.cons(head, tail).contains(x)
                        pred(f(x))
                    }
                }
                map[T, U](tail, f).filter(pred) = map[T, U](tail, f)
                List.cons(head, tail).contains(head)
                pred(f(head))
                map[T, U](List.cons(head, tail), f) = List.cons(f(head), map[T, U](tail, f))
                List.cons(f(head), map[T, U](tail, f)).filter(pred) =
                    List.cons(f(head), map[T, U](tail, f).filter(pred))
                map[T, U](List.cons(head, tail), f).filter(pred) =
                    List.cons(f(head), map[T, U](tail, f))
                map[T, U](List.cons(head, tail), f).filter(pred) =
                    map[T, U](List.cons(head, tail), f)
            }
            r(List.cons(head, tail))
        }
    }

    List.induction(function(xs: List[T]) { r(xs) })
    forall(xs: List[T]) {
        r(xs)
    }
    r(items)
}

/// Filtering before mapping has the expected membership criterion.
theorem map_filter_contains_iff[T, U](items: List[T], f: T -> U, pred: T -> Bool, item: U) {
    map[T, U](items.filter(pred), f).contains(item) = exists(x: T) {
        items.contains(x) and pred(x) and f(x) = item
    }
} by {
    if map[T, U](items.filter(pred), f).contains(item) {
        map_contains(items.filter(pred), f, item)
        let x: T satisfy {
            items.filter(pred).contains(x) and f(x) = item
        }
        filter_contained_by_and(items, pred, x)
        items.contains(x)
        pred(x)
        items.contains(x) and pred(x) and f(x) = item
        exists(y: T) {
            items.contains(y) and pred(y) and f(y) = item
        }
    }
    if exists(x: T) { items.contains(x) and pred(x) and f(x) = item } {
        let x: T satisfy {
            items.contains(x) and pred(x) and f(x) = item
        }
        filter_contains_and(items, pred, x)
        items.filter(pred).contains(x)
        map_contains_of_contains(items.filter(pred), f, x)
        map[T, U](items.filter(pred), f).contains(f(x))
        map[T, U](items.filter(pred), f).contains(item)
    }
}
