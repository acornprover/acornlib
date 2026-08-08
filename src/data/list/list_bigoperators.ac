from algebra.comm_monoid import CommMonoid
from list import List, map, product
from data.list.list_map_basic import map_nil, map_cons

/// Product induction for the current commutative list-product surface.
theorem prod_induction[A: CommMonoid](items: List[A], p: A -> Bool) {
    (forall(a: A, b: A) { p(a) and p(b) implies p(a * b) }) and
    p(A.1) and
    forall(x: A) { items.contains(x) implies p(x) } implies
    p(product[A](items))
} by {
    if (forall(a: A, b: A) { p(a) and p(b) implies p(a * b) }) and
        p(A.1) and
        forall(x: A) { items.contains(x) implies p(x) } {

        define q(xs: List[A]) -> Bool {
            forall(x: A) { xs.contains(x) implies p(x) } implies p(product[A](xs))
        }

        if forall(x: A) { List.nil[A].contains(x) implies p(x) } {
            product[A](List.nil[A]) = A.1
            p(A.1)
            p(product[A](List.nil[A]))
        }
        q(List.nil[A])

        forall(head: A, tail: List[A]) {
            if q(tail) {
                if forall(x: A) { List.cons(head, tail).contains(x) implies p(x) } {
                    List.cons(head, tail).contains(head)
                    p(head)
                    forall(x: A) {
                        if tail.contains(x) {
                            List.cons(head, tail).contains(x)
                            p(x)
                        }
                    }
                    forall(x: A) { tail.contains(x) implies p(x) }
                    p(product[A](tail))
                    p(head) and p(product[A](tail))
                    p(head * product[A](tail))
                    product[A](List.cons(head, tail)) = head * product[A](tail)
                    p(product[A](List.cons(head, tail)))
                }
                q(List.cons(head, tail))
            }
        }

        List.induction(function(xs: List[A]) { q(xs) })
        q(items)
        p(product[A](items))
    }
}

/// The product of a mapped constant-one list is one.
theorem prod_map_one[T, A: CommMonoid](items: List[T]) {
    product[A](map[T, A](items, function(x: T) { A.1 })) = A.1
} by {
    define p(xs: List[T]) -> Bool {
        product[A](map[T, A](xs, function(x: T) { A.1 })) = A.1
    }

    map_nil[T, A](function(x: T) { A.1 })
    product[A](map[T, A](List.nil[T], function(x: T) { A.1 })) = A.1
    p(List.nil[T])

    forall(head: T, tail: List[T]) {
        if p(tail) {
            map_cons(head, tail, function(x: T) { A.1 })
            map[T, A](List.cons(head, tail), function(x: T) { A.1 }) =
                List.cons(A.1, map[T, A](tail, function(x: T) { A.1 }))
            product[A](List.cons(A.1, map[T, A](tail, function(x: T) { A.1 }))) =
                A.1 * product[A](map[T, A](tail, function(x: T) { A.1 }))
            product[A](map[T, A](tail, function(x: T) { A.1 })) = A.1
            A.1 * A.1 = A.1
            product[A](map[T, A](List.cons(head, tail), function(x: T) { A.1 })) = A.1
            p(List.cons(head, tail))
        }
    }

    List.induction(function(xs: List[T]) { p(xs) })
    p(items)
}
