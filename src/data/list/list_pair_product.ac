from data.finite.finite_fiber_partition import locally_injective_map_is_unique
from data.basic.functions import compose, is_injective_fn
from list import List, map, add_contains_left, add_contains_or, add_contains_right,
    add_length, map_filter_length_of_pointwise, permutation_preserves_length,
    range_contains_iff_lt, range_is_unique, sum, unique_implies_tail_unique,
    unique_list_sum, unique_same_contains_filter_imp_permutation,
    injective_map_is_unique, is_permutation, map_contains, map_contains_of_contains,
    map_add, map_length, map_map, scalar_mul, sum_add, sum_scalar_mul
from nat import Nat, add_cancels_left, add_mod, add_zero_left, distrib_right,
    lt_and_lte, lt_diff, lte_add_left, lte_imp_not_lt, lte_mul, lte_trans,
    mul_suc_right, mul_zero_right, trichotomy
from pair import Pair, pair_eta

numerals Nat

/// The pair with fixed first component and variable second component.
define pair_with_left[T, U](x: T, y: U) -> Pair[T, U] {
    Pair.new(x, y)
}

/// The map that pairs a fixed left component with its input.
define pair_with_left_fn[T, U](x: T) -> U -> Pair[T, U] {
    function(y: U) { pair_with_left(x, y) }
}

/// The list of pairs obtained by pairing a fixed element with every element of a list.
define list_pair_with_left[T, U](x: T, right: List[U]) -> List[Pair[T, U]] {
    map[U, Pair[T, U]](right, pair_with_left_fn(x))
}

/// The Cartesian product of two lists, preserving the order of the left list first.
define list_pair_product[T, U](left: List[T], right: List[U]) -> List[Pair[T, U]] {
    match left {
        List.nil {
            List.nil[Pair[T, U]]
        }
        List.cons(head, tail) {
            list_pair_with_left(head, right) + list_pair_product(tail, right)
        }
    }
}

/// If the right component occurs in a right list, its pair with a fixed left
/// component occurs in the corresponding row of the list product.
theorem list_pair_with_left_contains_of_contains[T, U](x: T, right: List[U], y: U) {
    right.contains(y) implies list_pair_with_left(x, right).contains(Pair.new(x, y))
} by {
    if right.contains(y) {
        map_contains_of_contains[U, Pair[T, U]](right, pair_with_left_fn(x), y)
        list_pair_with_left(x, right).contains(pair_with_left(x, y))
        list_pair_with_left(x, right).contains(Pair.new(x, y))
    }
}

/// A row of the list product is contained in the full product of a nonempty
/// left list headed by that row's left component.
theorem list_pair_product_contains_left_row[T, U](head: T, tail: List[T],
    right: List[U], p: Pair[T, U]) {
    list_pair_with_left(head, right).contains(p) implies
    list_pair_product(List.cons(head, tail), right).contains(p)
} by {
    if list_pair_with_left(head, right).contains(p) {
        add_contains_left(list_pair_with_left(head, right), list_pair_product(tail, right), p)
        list_pair_product(List.cons(head, tail), right).contains(p)
    }
}

/// The tail product is contained in the full product of a nonempty left list.
theorem list_pair_product_contains_tail_product[T, U](head: T, tail: List[T],
    right: List[U], p: Pair[T, U]) {
    list_pair_product(tail, right).contains(p) implies
    list_pair_product(List.cons(head, tail), right).contains(p)
} by {
    if list_pair_product(tail, right).contains(p) {
        add_contains_right(list_pair_with_left(head, right), list_pair_product(tail, right), p)
        list_pair_product(List.cons(head, tail), right).contains(p)
    }
}

/// If two elements occur in the two factor lists, their pair occurs in the list product.
theorem list_pair_product_contains_of_contains[T, U](left: List[T], right: List[U],
    x: T, y: U) {
    left.contains(x) and right.contains(y) implies
    list_pair_product(left, right).contains(Pair.new(x, y))
} by {
    define p(items: List[T]) -> Bool {
        items.contains(x) and right.contains(y) implies
        list_pair_product(items, right).contains(Pair.new(x, y))
    }

    p(List.nil[T])

    forall(head: T, tail: List[T]) {
        if p(tail) {
            if List.cons(head, tail).contains(x) and right.contains(y) {
                if head = x {
                    list_pair_with_left_contains_of_contains(head, right, y)
                    list_pair_product_contains_left_row(head, tail, right, Pair.new(x, y))
                    list_pair_product(List.cons(head, tail), right).contains(Pair.new(x, y))
                } else {
                    list_pair_product_contains_tail_product(head, tail, right, Pair.new(x, y))
                    list_pair_product(List.cons(head, tail), right).contains(Pair.new(x, y))
                }
            }
            p(List.cons(head, tail))
        }
    }
    List.induction(function(items: List[T]) { p(items) })
    p(left)
}

/// Membership in a fixed-left product row determines the first projection.
theorem list_pair_with_left_contains_imp_first[T, U](x: T, right: List[U], p: Pair[T, U]) {
    list_pair_with_left(x, right).contains(p) implies p.first = x
} by {
    if list_pair_with_left(x, right).contains(p) {
        map_contains[U, Pair[T, U]](right, pair_with_left_fn(x), p)
        let y: U satisfy {
            right.contains(y) and pair_with_left_fn(x, y) = p
        }
        Pair.new(x, y).first = x
        p.first = x
    }
}

/// Membership in a fixed-left product row gives membership of the second
/// projection in the right list.
theorem list_pair_with_left_contains_imp_second[T, U](x: T, right: List[U], p: Pair[T, U]) {
    list_pair_with_left(x, right).contains(p) implies right.contains(p.second)
} by {
    if list_pair_with_left(x, right).contains(p) {
        map_contains[U, Pair[T, U]](right, pair_with_left_fn(x), p)
        let y: U satisfy {
            right.contains(y) and pair_with_left_fn(x, y) = p
        }
        pair_with_left_fn(x, y) = pair_with_left(x, y)
        Pair.new(x, y).second = y
        p.second = y
        right.contains(p.second)
    }
}

/// Pairing with a fixed left component is injective in the right component.
theorem pair_with_left_fn_injective[T, U](x: T) {
    is_injective_fn(pair_with_left_fn[T, U](x))
} by {
    forall(y1: U, y2: U) {
        if pair_with_left_fn(x, y1) = pair_with_left_fn(x, y2) {
            pair_with_left_fn(x, y1) = pair_with_left(x, y1)
            pair_with_left_fn(x, y2) = pair_with_left(x, y2)
            Pair.new(x, y1).second = y1
            Pair.new(x, y2).second = y2
            y1 = y2
        }
    }
}

/// A fixed-left product row preserves uniqueness of the right list.
theorem list_pair_with_left_unique[T, U](x: T, right: List[U]) {
    right.is_unique implies list_pair_with_left(x, right).is_unique
} by {
    if right.is_unique {
        pair_with_left_fn_injective[T, U](x)
        injective_map_is_unique[U, Pair[T, U]](right, pair_with_left_fn(x))
        list_pair_with_left(x, right).is_unique
    }
}

/// Membership of an explicit pair in a list Cartesian product gives membership
/// of its first component in the left list.
theorem list_pair_product_contains_pair_imp_left[T, U](left: List[T], right: List[U],
    x: T, y: U) {
    list_pair_product(left, right).contains(Pair.new(x, y)) implies left.contains(x)
} by {
    define q(items: List[T]) -> Bool {
        list_pair_product(items, right).contains(Pair.new(x, y)) implies items.contains(x)
    }

    q(List.nil[T])

    forall(head: T, tail: List[T]) {
        if q(tail) {
            if list_pair_product(List.cons(head, tail), right).contains(Pair.new(x, y)) {
                add_contains_or(list_pair_with_left(head, right), list_pair_product(tail, right),
                    Pair.new(x, y))
                if list_pair_with_left(head, right).contains(Pair.new(x, y)) {
                    list_pair_with_left_contains_imp_first(head, right, Pair.new(x, y))
                    List.cons(head, tail).contains(x)
                } else {
                    list_pair_product(tail, right).contains(Pair.new(x, y))
                    tail.contains(x)
                    List.cons(head, tail).contains(x)
                }
            }
            q(List.cons(head, tail))
        }
    }
    List.induction(function(items: List[T]) { q(items) })
    forall(items: List[T]) {
        q(items)
    }
}

/// Membership of an explicit pair in a list Cartesian product gives membership
/// of its second component in the right list.
theorem list_pair_product_contains_pair_imp_right[T, U](left: List[T], right: List[U],
    x: T, y: U) {
    list_pair_product(left, right).contains(Pair.new(x, y)) implies right.contains(y)
} by {
    define q(items: List[T]) -> Bool {
        list_pair_product(items, right).contains(Pair.new(x, y)) implies right.contains(y)
    }

    q(List.nil[T])

    forall(head: T, tail: List[T]) {
        if q(tail) {
            if list_pair_product(List.cons(head, tail), right).contains(Pair.new(x, y)) {
                add_contains_or(list_pair_with_left(head, right), list_pair_product(tail, right),
                    Pair.new(x, y))
                if list_pair_with_left(head, right).contains(Pair.new(x, y)) {
                    list_pair_with_left_contains_imp_second(head, right, Pair.new(x, y))
                    right.contains(y)
                } else {
                    list_pair_product(tail, right).contains(Pair.new(x, y))
                    right.contains(y)
                }
            }
            q(List.cons(head, tail))
        }
    }
    List.induction(function(items: List[T]) { q(items) })
    forall(items: List[T]) {
        q(items)
    }
}

/// Membership in a list Cartesian product gives membership of the first
/// projection in the left list.
theorem list_pair_product_contains_imp_left[T, U](left: List[T], right: List[U], p: Pair[T, U]) {
    list_pair_product(left, right).contains(p) implies left.contains(p.first)
} by {
    if list_pair_product(left, right).contains(p) {
        pair_eta(p)
        list_pair_product_contains_pair_imp_left(left, right, p.first, p.second)
        left.contains(p.first)
    }
}

/// Membership in a list Cartesian product gives membership of the second
/// projection in the right list.
theorem list_pair_product_contains_imp_right[T, U](left: List[T], right: List[U], p: Pair[T, U]) {
    list_pair_product(left, right).contains(p) implies right.contains(p.second)
} by {
    if list_pair_product(left, right).contains(p) {
        pair_eta(p)
        list_pair_product_contains_pair_imp_right(left, right, p.first, p.second)
        right.contains(p.second)
    }
}

/// Rows with different fixed-left values are disjoint.
theorem list_pair_with_left_disjoint_of_ne[T, U](x: T, z: T, right: List[U]) {
    x != z implies forall(p: Pair[T, U]) {
        not (list_pair_with_left(x, right).contains(p) and list_pair_with_left(z, right).contains(p))
    }
} by {
    if x != z {
        forall(p: Pair[T, U]) {
            if list_pair_with_left(x, right).contains(p) and list_pair_with_left(z, right).contains(p) {
                list_pair_with_left_contains_imp_first(x, right, p)
                list_pair_with_left_contains_imp_first(z, right, p)
                false
            }
        }
    }
}

/// If a left value is absent from a left list, then its product row is disjoint
/// from the product over that left list.
theorem list_pair_with_left_disjoint_product_of_not_contains[T, U](x: T, left: List[T],
    right: List[U]) {
    not left.contains(x) implies forall(p: Pair[T, U]) {
        not (list_pair_with_left(x, right).contains(p) and list_pair_product(left, right).contains(p))
    }
} by {
    if not left.contains(x) {
        forall(p: Pair[T, U]) {
            if list_pair_with_left(x, right).contains(p) and list_pair_product(left, right).contains(p) {
                list_pair_with_left_contains_imp_first(x, right, p)
                list_pair_product_contains_imp_left(left, right, p)
                false
            }
        }
    }
}

/// The Cartesian product of two unique lists is unique.
theorem list_pair_product_unique[T, U](left: List[T], right: List[U]) {
    left.is_unique and right.is_unique implies list_pair_product(left, right).is_unique
} by {
    define q(items: List[T]) -> Bool {
        items.is_unique and right.is_unique implies list_pair_product(items, right).is_unique
    }

    if List.nil[T].is_unique and right.is_unique {
        List.nil[Pair[T, U]].is_unique
        list_pair_product(List.nil[T], right).is_unique
    }
    q(List.nil[T])

    forall(head: T, tail: List[T]) {
        if q(tail) {
            if List.cons(head, tail).is_unique and right.is_unique {
                unique_implies_tail_unique(head, tail)
                tail.is_unique
                q(tail) = (tail.is_unique and right.is_unique implies
                    list_pair_product(tail, right).is_unique)
                tail.is_unique and right.is_unique implies list_pair_product(tail, right).is_unique
                list_pair_product(tail, right).is_unique

                list_pair_with_left_unique(head, right)
                list_pair_with_left(head, right).is_unique

                if tail.contains(head) {
                    List.cons(head, tail).unique = tail.unique
                    List.cons(head, tail).unique = List.cons(head, tail)
                    tail.unique = List.cons(head, tail)
                    tail.unique.length <= tail.length
                    List.cons(head, tail).length = tail.length.suc
                    tail.length.suc <= tail.length
                    false
                }
                list_pair_with_left_disjoint_product_of_not_contains(head, tail, right)
                forall(p: Pair[T, U]) {
                    not (list_pair_with_left(head, right).contains(p) and
                        list_pair_product(tail, right).contains(p))
                }
                unique_list_sum(list_pair_with_left(head, right), list_pair_product(tail, right))
                (list_pair_with_left(head, right) + list_pair_product(tail, right)).is_unique
                list_pair_product(List.cons(head, tail), right).is_unique
            }
            q(List.cons(head, tail))
        }
    }
    List.induction(function(items: List[T]) { q(items) })
    q(left)
}

/// A fixed-left product row has the same length as the right list.
theorem list_pair_with_left_length[T, U](x: T, right: List[U]) {
    list_pair_with_left(x, right).length = right.length
} by {
    map_length[U, Pair[T, U]](right, pair_with_left_fn(x))
}

/// Consing a left element adds one fixed-left row to the list Cartesian product.
theorem list_pair_product_cons_length[T, U](head: T, tail: List[T], right: List[U]) {
    list_pair_product(List.cons(head, tail), right).length =
    right.length + list_pair_product(tail, right).length
} by {
    let row = list_pair_with_left(head, right)
    let rest = list_pair_product(tail, right)
    add_length(row, rest)
    list_pair_with_left_length(head, right)
}

/// The length of a list Cartesian product is the right length times the left length.
theorem list_pair_product_length[T, U](left: List[T], right: List[U]) {
    list_pair_product(left, right).length = right.length * left.length
} by {
    define q(items: List[T]) -> Bool {
        list_pair_product(items, right).length = right.length * items.length
    }

    list_pair_product(List.nil[T], right) = List.nil[Pair[T, U]]
    List.nil[Pair[T, U]].length = Nat.0
    List.nil[T].length = Nat.0
    mul_zero_right(right.length)
    right.length * Nat.0 = Nat.0
    q(List.nil[T])

    forall(head: T, tail: List[T]) {
        if q(tail) {
            list_pair_product_cons_length(head, tail, right)
            list_pair_product(tail, right).length = right.length * tail.length
            mul_suc_right(right.length, tail.length)
            q(List.cons(head, tail))
        }
    }
    List.induction(function(items: List[T]) { q(items) })
    q(left)
}

/// Filtering a cons whose head satisfies the predicate keeps that head.
lemma list_filter_cons_of_true[T](head: T, tail: List[T], pred: T -> Bool) {
    pred(head) implies List.cons(head, tail).filter(pred) = List.cons(head, tail.filter(pred))
} by {
    if pred(head) {
        List.cons(head, tail).filter(pred) = List.cons(head, tail.filter(pred))
    }
}

/// Filtering a cons whose head does not satisfy the predicate skips that head.
lemma list_filter_cons_of_false[T](head: T, tail: List[T], pred: T -> Bool) {
    not pred(head) implies List.cons(head, tail).filter(pred) = tail.filter(pred)
} by {
    if not pred(head) {
        List.cons(head, tail).filter(pred) = tail.filter(pred)
    }
}

/// The length of a filtered concatenation is the sum of the filtered lengths.
lemma list_filter_add_length[T](left: List[T], right: List[T], pred: T -> Bool) {
    (left + right).filter(pred).length = left.filter(pred).length + right.filter(pred).length
} by {
    define q(items: List[T]) -> Bool {
        (items + right).filter(pred).length = items.filter(pred).length + right.filter(pred).length
    }

    q(List.nil[T])
    forall(head: T, tail: List[T]) {
        if q(tail) {
            if pred(head) {
                list_filter_cons_of_true(head, tail + right, pred)
                list_filter_cons_of_true(head, tail, pred)
                List.cons(head, tail).filter(pred).length = tail.filter(pred).length.suc
                (List.cons(head, tail) + right).filter(pred).length =
                    List.cons(head, tail).filter(pred).length + right.filter(pred).length
            }
            if not pred(head) {
                list_filter_cons_of_false(head, tail + right, pred)
                list_filter_cons_of_false(head, tail, pred)
                (List.cons(head, tail) + right).filter(pred).length =
                    List.cons(head, tail).filter(pred).length + right.filter(pred).length
            }
            q(List.cons(head, tail))
        }
    }

    List.induction(function(items: List[T]) { q(items) })
}

/// A fixed-left product row filtered by a pair predicate has the same length as
/// filtering the right list by the corresponding unary predicate.
theorem list_pair_with_left_filter_length[T, U](x: T, right: List[U], pred: Pair[T, U] -> Bool) {
    list_pair_with_left(x, right).filter(pred).length =
    right.filter(function(y: U) { pred(Pair.new(x, y)) }).length
} by {
    forall(y: U) {
        if right.contains(y) {
            pred(pair_with_left_fn(x, y)) = pred(Pair.new(x, y))
        }
    }
    map_filter_length_of_pointwise[U, Pair[T, U]](right, pair_with_left_fn(x), pred,
        function(y: U) { pred(Pair.new(x, y)) })
    map[U, Pair[T, U]](right, pair_with_left_fn(x)).filter(pred).length =
        right.filter(function(y: U) { pred(Pair.new(x, y)) }).length
}

/// The row contribution to a filtered list product.
define list_pair_product_row_filter_length_fn[T, U](right: List[U], pred: Pair[T, U] -> Bool) -> (T -> Nat) {
    function(x: T) { list_pair_with_left(x, right).filter(pred).length }
}

/// The filtered length of a list product is the sum of the filtered row lengths.
theorem list_pair_product_filter_length_sum_rows[T, U](left: List[T], right: List[U],
    pred: Pair[T, U] -> Bool) {
    list_pair_product(left, right).filter(pred).length =
    sum[Nat](map[T, Nat](left, list_pair_product_row_filter_length_fn(right, pred)))
} by {
    define q(items: List[T]) -> Bool {
        list_pair_product(items, right).filter(pred).length =
        sum[Nat](map[T, Nat](items, list_pair_product_row_filter_length_fn(right, pred)))
    }

    list_pair_product(List.nil[T], right) = List.nil[Pair[T, U]]
    map[T, Nat](List.nil[T], list_pair_product_row_filter_length_fn(right, pred)) = List.nil[Nat]
    list_pair_product(List.nil[T], right).filter(pred) = List.nil[Pair[T, U]]
    List.nil[Pair[T, U]].length = Nat.0
    list_pair_product(List.nil[T], right).filter(pred).length = Nat.0
    sum[Nat](map[T, Nat](List.nil[T], list_pair_product_row_filter_length_fn(right, pred))) = Nat.0
    q(List.nil[T])

    forall(head: T, tail: List[T]) {
        if q(tail) {
            let row = list_pair_with_left(head, right)
            let rest = list_pair_product(tail, right)
            list_filter_add_length(row, rest, pred)
            rest.filter(pred).length =
                sum[Nat](map[T, Nat](tail, list_pair_product_row_filter_length_fn(right, pred)))
            list_pair_product_row_filter_length_fn(right, pred)(head) = row.filter(pred).length
            sum[Nat](List.cons(row.filter(pred).length,
                map[T, Nat](tail, list_pair_product_row_filter_length_fn(right, pred)))) =
                row.filter(pred).length +
                    sum[Nat](map[T, Nat](tail, list_pair_product_row_filter_length_fn(right, pred)))
            list_pair_product(List.cons(head, tail), right).filter(pred).length =
                sum[Nat](map[T, Nat](List.cons(head, tail), list_pair_product_row_filter_length_fn(right, pred)))
            q(List.cons(head, tail))
        }
    }
    List.induction(function(items: List[T]) { q(items) })
    q(left)
}

/// The product predicate that splits through the two projections of a pair.
define list_pair_product_split_filter_pred[T, U](lp: T -> Bool, rp: U -> Bool) -> (Pair[T, U] -> Bool) {
    function(p: Pair[T, U]) { lp(p.first) and rp(p.second) }
}

/// A selected fixed-left row contributes exactly the selected right length.
theorem list_pair_with_left_split_filter_length_of_left[T, U](x: T, right: List[U],
    lp: T -> Bool, rp: U -> Bool) {
    lp(x) implies list_pair_with_left(x, right).filter(
        list_pair_product_split_filter_pred(lp, rp)).length = right.filter(rp).length
} by {
    define q(items: List[U]) -> Bool {
        list_pair_with_left(x, items).filter(
            list_pair_product_split_filter_pred(lp, rp)).length = items.filter(rp).length
    }

    if lp(x) {
        list_pair_with_left(x, List.nil[U]) = List.nil[Pair[T, U]]
        List.nil[Pair[T, U]].filter(list_pair_product_split_filter_pred(lp, rp)) =
            List.nil[Pair[T, U]]
        List.nil[U].filter(rp) = List.nil[U]
        List.nil[Pair[T, U]].length = Nat.0
        List.nil[U].length = Nat.0
        q(List.nil[U])

        forall(head: U, tail: List[U]) {
            if q(tail) {
                let row_tail = list_pair_with_left(x, tail)
                let row_head = pair_with_left_fn(x, head)
                list_pair_with_left(x, List.cons(head, tail)) =
                    List.cons(row_head, row_tail)
                pair_with_left_fn(x, head) = pair_with_left(x, head)
                pair_with_left(x, head) = Pair.new(x, head)
                row_head = Pair.new(x, head)
                row_head.first = x
                row_head.second = head
                row_tail.filter(list_pair_product_split_filter_pred(lp, rp)).length = tail.filter(rp).length
                if rp(head) {
                    list_pair_product_split_filter_pred(lp, rp)(row_head)
                    list_filter_cons_of_true(row_head, row_tail,
                        list_pair_product_split_filter_pred(lp, rp))
                    list_filter_cons_of_true(head, tail, rp)
                    List.cons(row_head, row_tail).filter(
                        list_pair_product_split_filter_pred(lp, rp)).length =
                        row_tail.filter(list_pair_product_split_filter_pred(lp, rp)).length.suc
                    List.cons(head, tail).filter(rp).length = tail.filter(rp).length.suc
                    list_pair_with_left(x, List.cons(head, tail)).filter(
                        list_pair_product_split_filter_pred(lp, rp)).length =
                        List.cons(head, tail).filter(rp).length
                }
                if not rp(head) {
                    if list_pair_product_split_filter_pred(lp, rp)(row_head) {
                        rp(head)
                        false
                    }
                    list_filter_cons_of_false(row_head, row_tail,
                        list_pair_product_split_filter_pred(lp, rp))
                    list_filter_cons_of_false(head, tail, rp)
                    list_pair_with_left(x, List.cons(head, tail)).filter(
                        list_pair_product_split_filter_pred(lp, rp)).length =
                        List.cons(head, tail).filter(rp).length
                }
                q(List.cons(head, tail))
            }
        }
        List.induction(function(items: List[U]) { q(items) })
        q(right)
    }
}

/// An unselected fixed-left row contributes no elements to a split product filter.
theorem list_pair_with_left_split_filter_length_of_not_left[T, U](x: T, right: List[U],
    lp: T -> Bool, rp: U -> Bool) {
    not lp(x) implies list_pair_with_left(x, right).filter(
        list_pair_product_split_filter_pred(lp, rp)).length = Nat.0
} by {
    define q(items: List[U]) -> Bool {
        list_pair_with_left(x, items).filter(
            list_pair_product_split_filter_pred(lp, rp)).length = Nat.0
    }

    if not lp(x) {
        list_pair_with_left(x, List.nil[U]) = List.nil[Pair[T, U]]
        List.nil[Pair[T, U]].filter(list_pair_product_split_filter_pred(lp, rp)) =
            List.nil[Pair[T, U]]
        List.nil[Pair[T, U]].length = Nat.0
        q(List.nil[U])

        forall(head: U, tail: List[U]) {
            if q(tail) {
                let row_tail = list_pair_with_left(x, tail)
                let row_head = pair_with_left_fn(x, head)
                list_pair_with_left(x, List.cons(head, tail)) =
                    List.cons(row_head, row_tail)
                pair_with_left_fn(x, head) = pair_with_left(x, head)
                pair_with_left(x, head) = Pair.new(x, head)
                row_head = Pair.new(x, head)
                row_head.first = x
                if list_pair_product_split_filter_pred(lp, rp)(row_head) {
                    lp(row_head.first)
                    false
                }
                row_tail.filter(list_pair_product_split_filter_pred(lp, rp)).length = Nat.0
                list_filter_cons_of_false(row_head, row_tail,
                    list_pair_product_split_filter_pred(lp, rp))
                list_pair_with_left(x, List.cons(head, tail)).filter(
                    list_pair_product_split_filter_pred(lp, rp)).length = Nat.0
                q(List.cons(head, tail))
            }
        }
        List.induction(function(items: List[U]) { q(items) })
        q(right)
    }
}

/// For a product predicate that splits by projections, the filtered product length factors.
theorem list_pair_product_filter_length_product[T, U](left: List[T], right: List[U],
    lp: T -> Bool, rp: U -> Bool) {
    list_pair_product(left, right).filter(list_pair_product_split_filter_pred(lp, rp)).length =
    right.filter(rp).length * left.filter(lp).length
} by {
    let pred = list_pair_product_split_filter_pred(lp, rp)
    let c = right.filter(rp).length
    define q(items: List[T]) -> Bool {
        list_pair_product(items, right).filter(pred).length = c * items.filter(lp).length
    }

    list_pair_product(List.nil[T], right) = List.nil[Pair[T, U]]
    List.nil[Pair[T, U]].filter(pred) = List.nil[Pair[T, U]]
    List.nil[T].filter(lp) = List.nil[T]
    list_pair_product(List.nil[T], right).filter(pred).length = Nat.0
    List.nil[T].filter(lp).length = Nat.0
    mul_zero_right(c)
    c * List.nil[T].filter(lp).length = Nat.0
    q(List.nil[T])

    forall(head: T, tail: List[T]) {
        if q(tail) {
            let row = list_pair_with_left(head, right)
            let rest = list_pair_product(tail, right)
            list_filter_add_length(row, rest, pred)
            rest.filter(pred).length = c * tail.filter(lp).length
            if lp(head) {
                list_pair_with_left_split_filter_length_of_left(head, right, lp, rp)
                list_pair_with_left(head, right).filter(pred).length = c
                list_filter_cons_of_true(head, tail, lp)
                List.cons(head, tail).filter(lp).length = tail.filter(lp).length.suc
                mul_suc_right(c, tail.filter(lp).length)
                list_pair_product(List.cons(head, tail), right).filter(pred).length =
                    c * List.cons(head, tail).filter(lp).length
            }
            if not lp(head) {
                list_pair_with_left_split_filter_length_of_not_left(head, right, lp, rp)
                list_filter_cons_of_false(head, tail, lp)
                add_zero_left(c * tail.filter(lp).length)
                list_pair_product(List.cons(head, tail), right).filter(pred).length =
                    c * List.cons(head, tail).filter(lp).length
            }
            q(List.cons(head, tail))
        }
    }
    List.induction(function(items: List[T]) { q(items) })
    q(left)
    list_pair_product(left, right).filter(pred).length = c * left.filter(lp).length
}

/// The value `f(x) * g(y)` attached to a pair `(x, y)`.
define pair_product_map_value[T, U](f: T -> Nat, g: U -> Nat) -> (Pair[T, U] -> Nat) {
    function(p: Pair[T, U]) { f(p.first) * g(p.second) }
}

/// Summing `f(x) * g(y)` over a fixed-left row factors out `f(x)`.
theorem list_pair_with_left_map_mul_sum[T, U](x: T, right: List[U], f: T -> Nat,
    g: U -> Nat) {
    sum(map(list_pair_with_left(x, right), pair_product_map_value(f, g))) =
        f(x) * sum(map(right, g))
} by {
    let h = pair_product_map_value(f, g)
    list_pair_with_left(x, right) = map(right, pair_with_left_fn[T, U](x))
    map_map[U, Pair[T, U], Nat](right, pair_with_left_fn[T, U](x), h)
    map(map(right, pair_with_left_fn[T, U](x)), h) =
        map(right, compose(h, pair_with_left_fn[T, U](x)))
    forall(y: U) {
        compose(h, pair_with_left_fn[T, U](x), y) =
            h(pair_with_left_fn[T, U](x, y))
        pair_with_left_fn[T, U](x, y) = Pair.new(x, y)
        h(Pair.new(x, y)) = f(x) * g(y)
        scalar_mul(f(x), g(y)) = f(x) * g(y)
        compose(h, pair_with_left_fn[T, U](x), y) =
            compose(scalar_mul(f(x)), g, y)
    }
    map_map[U, Nat, Nat](right, g, scalar_mul(f(x)))
    sum_scalar_mul(f(x), map(right, g))
}

/// Summing `f(x) * g(y)` over a list Cartesian product factors into the
/// product of the two mapped sums.
theorem list_pair_product_map_mul_sum[T, U](left: List[T], right: List[U], f: T -> Nat,
    g: U -> Nat) {
    sum(map(list_pair_product(left, right), pair_product_map_value(f, g))) =
        sum(map(left, f)) * sum(map(right, g))
} by {
    let h = pair_product_map_value(f, g)
    define q(items: List[T]) -> Bool {
        sum(map(list_pair_product(items, right), h)) =
            sum(map(items, f)) * sum(map(right, g))
    }

    list_pair_product(List.nil[T], right) = List.nil[Pair[T, U]]
    map[Pair[T, U], Nat](List.nil[Pair[T, U]], h) = List.nil[Nat]
    map[T, Nat](List.nil[T], f) = List.nil[Nat]
    sum[Nat](List.nil[Nat]) = Nat.0
    sum[Nat](List.nil[Nat]) * sum(map(right, g)) = Nat.0
    q(List.nil[T])

    forall(head: T, tail: List[T]) {
        if q(tail) {
            let row: List[Pair[T, U]] = list_pair_with_left(head, right)
            let rest: List[Pair[T, U]] = list_pair_product(tail, right)
            map_add[Pair[T, U], Nat](row, rest, h)
            sum_add(map(row, h), map(rest, h))
            list_pair_with_left_map_mul_sum(head, right, f, g)
            sum(map(rest, h)) = sum(map(tail, f)) * sum(map(right, g))
            distrib_right(f(head), sum(map(tail, f)), sum(map(right, g)))
            let mapped_tail: List[Nat] = map(tail, f)
            sum(List.cons(f(head), mapped_tail)) = f(head) + sum(mapped_tail)
            sum(map(list_pair_product(List.cons(head, tail), right), h)) =
                sum(map(List.cons(head, tail), f)) * sum(map(right, g))
            q(List.cons(head, tail))
        }
    }
    List.induction(function(items: List[T]) { q(items) })
    q(left)
}

/// The function that is `c` on elements satisfying `pred`, and zero elsewhere.
define filter_indicator_value[T](c: Nat, pred: T -> Bool) -> (T -> Nat) {
    function(x: T) { if pred(x) { c } else { Nat.0 } }
}

/// Summing a constant over the selected elements of a list gives that constant
/// times the length of the filtered list.
theorem sum_filter_indicator_value[T](items: List[T], c: Nat, pred: T -> Bool) {
    sum(map(items, filter_indicator_value(c, pred))) = c * items.filter(pred).length
} by {
    let f = filter_indicator_value(c, pred)
    define q(xs: List[T]) -> Bool {
        sum(map(xs, f)) = c * xs.filter(pred).length
    }

    map[T, Nat](List.nil[T], f) = List.nil[Nat]
    List.nil[T].filter(pred) = List.nil[T]
    List.nil[T].filter(pred).length = Nat.0
    sum[Nat](List.nil[Nat]) = Nat.0
    sum(map(List.nil[T], f)) = Nat.0
    mul_zero_right(c)
    c * Nat.0 = Nat.0
    c * List.nil[T].filter(pred).length = Nat.0
    sum(map(List.nil[T], f)) = c * List.nil[T].filter(pred).length
    q(List.nil[T])

    forall(head: T, tail: List[T]) {
        if q(tail) {
            map(List.cons(head, tail), f) = List.cons(f(head), map(tail, f))
            sum(List.cons(f(head), map(tail, f))) = f(head) + sum(map(tail, f))
            sum(map(List.cons(head, tail), f)) = f(head) + sum(map(tail, f))
            q(tail)
            sum(map(tail, f)) = c * tail.filter(pred).length
            if pred(head) {
                f(head) = c
                list_filter_cons_of_true(head, tail, pred)
                List.cons(head, tail).filter(pred).length = tail.filter(pred).length.suc
                mul_suc_right(c, tail.filter(pred).length)
                sum(map(List.cons(head, tail), f)) =
                    c * List.cons(head, tail).filter(pred).length
            }
            if not pred(head) {
                f(head) = Nat.0
                list_filter_cons_of_false(head, tail, pred)
                sum(map(List.cons(head, tail), f)) =
                    c * List.cons(head, tail).filter(pred).length
            }
            q(List.cons(head, tail))
        }
    }
    List.induction(function(xs: List[T]) { q(xs) })
    q(items)
}

/// Row-major encoding of a pair whose first component is a column below `m`
/// and whose second component is a row.
define row_encode(m: Nat) -> (Pair[Nat, Nat] -> Nat) {
    function(p: Pair[Nat, Nat]) { p.second * m + p.first }
}

/// The row-major values obtained from the `m` by `n` rectangle of ranges.
define rectangle_values(m: Nat, n: Nat) -> List[Nat] {
    map[Pair[Nat, Nat], Nat](list_pair_product[Nat, Nat](m.range, n.range), row_encode(m))
}

/// Bounded quotient-remainder decompositions over a nonzero modulus are unique.
theorem row_decomposition_unique(m: Nat, a: Nat, b: Nat, c: Nat, d: Nat) {
    m != Nat.0 and a < m and c < m and b * m + a = d * m + c implies a = c and b = d
} by {
    if m != Nat.0 and a < m and c < m and b * m + a = d * m + c {
        if b != d {
            trichotomy(b, d)
            if b < d {
                lt_diff(b, d)
                let e: Nat satisfy { b + e = d and e != Nat.0 }
                distrib_right(b, e, m)
                add_cancels_left(b * m, a, e * m + c)
                a = e * m + c
                lte_mul(m, e)
                lte_add_left(e * m, Nat.0, c)
                e * m <= e * m + c
                lte_trans(m, e * m, e * m + c)
                m <= e * m + c
                lte_imp_not_lt(m, a)
                false
            }
            if d < b {
                lt_diff(d, b)
                let e: Nat satisfy { d + e = b and e != Nat.0 }
                distrib_right(d, e, m)
                add_cancels_left(d * m, e * m + a, c)
                e * m + a = c
                lte_mul(m, e)
                lte_add_left(e * m, Nat.0, a)
                e * m <= e * m + a
                lte_trans(m, e * m, e * m + a)
                m <= e * m + a
                lte_imp_not_lt(m, c)
                false
            }
            false
        }
        add_cancels_left(b * m, a, c)
        a = c
        if not (a = c and b = d) {
            false
        }
        a = c and b = d
    }
}

/// Row-major encoding is injective on the rectangle `m.range × n.range`.
theorem row_encode_locally_injective(m: Nat, n: Nat) {
    m != Nat.0 implies forall(p: Pair[Nat, Nat], q: Pair[Nat, Nat]) {
        list_pair_product[Nat, Nat](m.range, n.range).contains(p) and
        list_pair_product[Nat, Nat](m.range, n.range).contains(q) and
        row_encode(m)(p) = row_encode(m)(q) implies p = q
    }
} by {
    if m != Nat.0 {
        forall(p: Pair[Nat, Nat], q: Pair[Nat, Nat]) {
            if list_pair_product[Nat, Nat](m.range, n.range).contains(p) and
                list_pair_product[Nat, Nat](m.range, n.range).contains(q) and
                row_encode(m)(p) = row_encode(m)(q) {
                list_pair_product_contains_imp_left[Nat, Nat](m.range, n.range, p)
                list_pair_product_contains_imp_right[Nat, Nat](m.range, n.range, p)
                list_pair_product_contains_imp_left[Nat, Nat](m.range, n.range, q)
                list_pair_product_contains_imp_right[Nat, Nat](m.range, n.range, q)
                range_contains_iff_lt(m, p.first)
                range_contains_iff_lt(m, q.first)
                p.first < m
                q.first < m
                p.second * m + p.first = q.second * m + q.first
                row_decomposition_unique(m, p.first, p.second, q.first, q.second)
                p.first = q.first
                p.second = q.second
                pair_eta(p)
                pair_eta(q)
                p = Pair.new(p.first, p.second)
                q = Pair.new(q.first, q.second)
                p = q
            }
        }
    }
}

/// Every value encoded by an `m` by `n` rectangle is below `m * n`.
theorem rectangle_values_contains_imp_lt(m: Nat, n: Nat, x: Nat) {
    rectangle_values(m, n).contains(x) implies x < m * n
} by {
    if rectangle_values(m, n).contains(x) {
        rectangle_values(m, n) =
            map[Pair[Nat, Nat], Nat](list_pair_product[Nat, Nat](m.range, n.range), row_encode(m))
        map_contains[Pair[Nat, Nat], Nat](list_pair_product[Nat, Nat](m.range, n.range),
            row_encode(m), x)
        let p: Pair[Nat, Nat] satisfy {
            list_pair_product[Nat, Nat](m.range, n.range).contains(p) and row_encode(m)(p) = x
        }
        list_pair_product_contains_imp_left[Nat, Nat](m.range, n.range, p)
        list_pair_product_contains_imp_right[Nat, Nat](m.range, n.range, p)
        m.range.contains(p.first)
        n.range.contains(p.second)
        range_contains_iff_lt(m, p.first)
        range_contains_iff_lt(n, p.second)
        p.first < m
        p.second < n
        p.second.suc <= n
        p.second.suc * m <= n * m
        p.second.suc * m = m + p.second * m
        m + p.second * m = p.second * m + m
        p.second * m + m <= n * m
        p.second * m + p.first < p.second * m + m
        lt_and_lte(p.second * m + p.first, p.second * m + m, n * m)
        p.second * m + p.first < n * m
        n * m = m * n
        row_encode(m)(p) = p.second * m + p.first
        x = p.second * m + p.first
        x < m * n
    }
}

/// Every value below `m*n` is encoded by a unique row and column in the rectangle.
theorem rectangle_values_contains_of_lt(m: Nat, n: Nat, x: Nat) {
    m != Nat.0 and x < m * n implies rectangle_values(m, n).contains(x)
} by {
    if m != Nat.0 and x < m * n {
        let col = x.mod(m)
        let row = x.div(m)
        add_mod(x, m)
        row * m + col = x
        col < m
        if not row < n {
            n <= row
            n * m <= row * m
            Nat.0 <= col
            lte_add_left(row * m, Nat.0, col)
            row * m + Nat.0 <= row * m + col
            row * m + Nat.0 = row * m
            row * m <= row * m + col
            lte_trans(n * m, row * m, row * m + col)
            n * m <= row * m + col
            n * m <= x
            m * n = n * m
            m * n <= x
            lte_imp_not_lt(m * n, x)
            false
        }
        row < n
        range_contains_iff_lt(m, col)
        range_contains_iff_lt(n, row)
        m.range.contains(col)
        n.range.contains(row)
        list_pair_product_contains_of_contains[Nat, Nat](m.range, n.range, col, row)
        list_pair_product[Nat, Nat](m.range, n.range).contains(Pair.new(col, row))
        Pair.new(col, row).first = col
        Pair.new(col, row).second = row
        row_encode(m)(Pair.new(col, row)) = row * m + col
        row_encode(m)(Pair.new(col, row)) = x
        map_contains_of_contains[Pair[Nat, Nat], Nat](list_pair_product[Nat, Nat](m.range, n.range),
            row_encode(m), Pair.new(col, row))
        map[Pair[Nat, Nat], Nat](list_pair_product[Nat, Nat](m.range, n.range),
            row_encode(m)).contains(row_encode(m)(Pair.new(col, row)))
        rectangle_values(m, n).contains(row_encode(m)(Pair.new(col, row)))
        rectangle_values(m, n).contains(x)
    }
}

/// The row-major rectangle values are exactly the initial interval `[0, m*n)`.
theorem rectangle_values_contains_iff(m: Nat, n: Nat, x: Nat) {
    m != Nat.0 implies rectangle_values(m, n).contains(x) = (x < m * n)
} by {
    if m != Nat.0 {
        if rectangle_values(m, n).contains(x) {
            rectangle_values_contains_imp_lt(m, n, x)
            x < m * n
        }
        if x < m * n {
            rectangle_values_contains_of_lt(m, n, x)
            rectangle_values(m, n).contains(x)
        }
        rectangle_values(m, n).contains(x) = (x < m * n)
    }
}

/// The row-major rectangle enumeration has no duplicates.
theorem rectangle_values_unique(m: Nat, n: Nat) {
    m != Nat.0 implies rectangle_values(m, n).is_unique
} by {
    if m != Nat.0 {
        range_is_unique(m)
        range_is_unique(n)
        m.range.is_unique
        n.range.is_unique
        list_pair_product_unique[Nat, Nat](m.range, n.range)
        list_pair_product[Nat, Nat](m.range, n.range).is_unique
        row_encode_locally_injective(m, n)
        forall(p: Pair[Nat, Nat], q: Pair[Nat, Nat]) {
            if list_pair_product[Nat, Nat](m.range, n.range).contains(p) and
                list_pair_product[Nat, Nat](m.range, n.range).contains(q) and
                row_encode(m)(p) = row_encode(m)(q) {
                p = q
            }
        }
        locally_injective_map_is_unique[Pair[Nat, Nat], Nat](
            list_pair_product[Nat, Nat](m.range, n.range), row_encode(m))
        map[Pair[Nat, Nat], Nat](list_pair_product[Nat, Nat](m.range, n.range), row_encode(m)).is_unique
        rectangle_values(m, n).is_unique
    }
}

/// The row-major rectangle enumeration and the initial range have the same elements.
theorem rectangle_values_same_contains_range(m: Nat, n: Nat, x: Nat) {
    m != Nat.0 implies rectangle_values(m, n).contains(x) = (m * n).range.contains(x)
} by {
    if m != Nat.0 {
        rectangle_values_contains_iff(m, n, x)
        rectangle_values(m, n).contains(x) = (x < m * n)
        range_contains_iff_lt(m * n, x)
        (m * n).range.contains(x) = (x < m * n)
        rectangle_values(m, n).contains(x) = (m * n).range.contains(x)
    }
}

/// Filtering the row-major rectangle enumeration gives a permutation of filtering the initial range.
theorem rectangle_values_filter_permutation_range_filter(m: Nat, n: Nat, pred: Nat -> Bool) {
    m != Nat.0 implies is_permutation(
        rectangle_values(m, n).filter(pred),
        (m * n).range.filter(pred))
} by {
    if m != Nat.0 {
        rectangle_values_unique(m, n)
        rectangle_values(m, n).is_unique
        range_is_unique(m * n)
        (m * n).range.is_unique
        forall(x: Nat) {
            rectangle_values_same_contains_range(m, n, x)
            rectangle_values(m, n).contains(x) = (m * n).range.contains(x)
        }
        unique_same_contains_filter_imp_permutation[Nat](rectangle_values(m, n), (m * n).range, pred)
        is_permutation(rectangle_values(m, n).filter(pred), (m * n).range.filter(pred))
    }
}

/// Filtering the row-major rectangle enumeration has the same length as filtering the initial range.
theorem rectangle_values_filter_length_range(m: Nat, n: Nat, pred: Nat -> Bool) {
    m != Nat.0 implies rectangle_values(m, n).filter(pred).length =
        (m * n).range.filter(pred).length
} by {
    if m != Nat.0 {
        rectangle_values_filter_permutation_range_filter(m, n, pred)
        is_permutation(rectangle_values(m, n).filter(pred), (m * n).range.filter(pred))
        permutation_preserves_length[Nat](rectangle_values(m, n).filter(pred),
            (m * n).range.filter(pred))
        rectangle_values(m, n).filter(pred).length = (m * n).range.filter(pred).length
    }
}

/// Predicate on rectangle coordinates obtained by applying a predicate to their row-major encoding.
define row_encode_filter_pred(m: Nat, pred: Nat -> Bool) -> (Pair[Nat, Nat] -> Bool) {
    function(p: Pair[Nat, Nat]) { pred(row_encode(m)(p)) }
}

/// Filtering rectangle values has the same length as filtering the source rectangle by the encoded predicate.
theorem rectangle_values_filter_length_pair_product(m: Nat, n: Nat, pred: Nat -> Bool) {
    rectangle_values(m, n).filter(pred).length =
        list_pair_product[Nat, Nat](m.range, n.range).filter(row_encode_filter_pred(m, pred)).length
} by {
    forall(p: Pair[Nat, Nat]) {
        if list_pair_product[Nat, Nat](m.range, n.range).contains(p) {
            row_encode_filter_pred(m, pred)(p) = pred(row_encode(m)(p))
        }
    }
    map_filter_length_of_pointwise[Pair[Nat, Nat], Nat](
        list_pair_product[Nat, Nat](m.range, n.range), row_encode(m), pred,
        row_encode_filter_pred(m, pred))
    map[Pair[Nat, Nat], Nat](list_pair_product[Nat, Nat](m.range, n.range), row_encode(m)).filter(pred).length =
        list_pair_product[Nat, Nat](m.range, n.range).filter(row_encode_filter_pred(m, pred)).length
    rectangle_values(m, n) =
        map[Pair[Nat, Nat], Nat](list_pair_product[Nat, Nat](m.range, n.range), row_encode(m))
    rectangle_values(m, n).filter(pred).length =
        list_pair_product[Nat, Nat](m.range, n.range).filter(row_encode_filter_pred(m, pred)).length
}

/// Filtering the source rectangle by an encoded predicate has the same length as filtering the initial range.
theorem row_encode_filter_length_range(m: Nat, n: Nat, pred: Nat -> Bool) {
    m != Nat.0 implies
    list_pair_product[Nat, Nat](m.range, n.range).filter(row_encode_filter_pred(m, pred)).length =
        (m * n).range.filter(pred).length
} by {
    if m != Nat.0 {
        rectangle_values_filter_length_pair_product(m, n, pred)
        rectangle_values(m, n).filter(pred).length =
            list_pair_product[Nat, Nat](m.range, n.range).filter(row_encode_filter_pred(m, pred)).length
        rectangle_values_filter_length_range(m, n, pred)
        rectangle_values(m, n).filter(pred).length = (m * n).range.filter(pred).length
        list_pair_product[Nat, Nat](m.range, n.range).filter(row_encode_filter_pred(m, pred)).length =
            (m * n).range.filter(pred).length
    }
}
