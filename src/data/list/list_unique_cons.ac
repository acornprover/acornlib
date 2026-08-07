from nat import Nat, lte_trans
from list import List, unique_length, unique_implies_tail_unique

numerals Nat

/// A list without repeats does not contain its own head again.
///
/// `is_unique` is `self.unique = self` rather than a structural property, so this is not
/// immediate. If the tail already contained the head, `unique` would drop the head and
/// return the tail's deduplication, which is too short to equal the original list. The
/// argument is therefore about length, not membership.
theorem unique_cons_head_not_in_tail[T](head: T, tail: List[T]) {
    List.cons(head, tail).is_unique implies not tail.contains(head)
} by {
    if List.cons(head, tail).is_unique {
        List.cons(head, tail).unique = List.cons(head, tail)
        if tail.contains(head) {
            List.cons(head, tail).unique = tail.unique
            tail.unique = List.cons(head, tail)
            tail.unique.length = List.cons(head, tail).length
            List.cons(head, tail).length = tail.length + Nat.1
            tail.unique.length = tail.length + Nat.1
            unique_length(tail)
            tail.unique.length <= tail.length
            tail.length + Nat.1 <= tail.length
            false
        }
        not tail.contains(head)
    }
}

/// The head of a list without repeats does not appear in its tail, as a membership fact.
theorem unique_cons_head_fresh[T](head: T, tail: List[T], item: T) {
    List.cons(head, tail).is_unique and tail.contains(item) implies head != item
} by {
    if List.cons(head, tail).is_unique and tail.contains(item) {
        unique_cons_head_not_in_tail(head, tail)
        not tail.contains(head)
        if head = item {
            tail.contains(head)
            false
        }
        head != item
    }
}

/// Both parts of the uniqueness of a prepended list.
theorem unique_cons_parts[T](head: T, tail: List[T]) {
    List.cons(head, tail).is_unique implies tail.is_unique and not tail.contains(head)
} by {
    if List.cons(head, tail).is_unique {
        unique_implies_tail_unique(head, tail)
        tail.is_unique
        unique_cons_head_not_in_tail(head, tail)
        not tail.contains(head)
    }
}
