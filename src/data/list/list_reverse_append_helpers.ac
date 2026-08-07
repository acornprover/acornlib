from list import List, reverse, reverse_involution

/// The reverse of the empty list is empty.
theorem reverse_nil[T] {
    reverse(List.nil[T]) = List.nil[T]
}

/// The reverse of a singleton list is itself.
theorem reverse_singleton[T](item: T) {
    reverse(List.singleton(item)) = List.singleton(item)
} by {
    List.nil[T] + List.singleton(item) = List.singleton(item)
}

/// Reversing a cons appends the head to the reversed tail.
theorem reverse_cons_eq_append[T](head: T, tail: List[T]) {
    reverse(List.cons(head, tail)) = reverse(tail) + List.singleton(head)
} by {
}

/// Reversing a list with one item appended puts that item at the front.
theorem reverse_append_item[T](items: List[T], item: T) {
    reverse(items.append(item)) = List.cons(item, reverse(items))
} by {
    let target = List.cons(item, reverse(items))
    reverse_cons_eq_append(item, reverse(items))
    reverse_involution(items)
    reverse_involution(target)
}

/// Reversing an explicit singleton append puts that item at the front.
theorem reverse_append_singleton[T](items: List[T], item: T) {
    reverse(items + List.singleton(item)) = List.cons(item, reverse(items))
} by {
    reverse_append_item(items, item)
}

/// If a reversed list is empty, then the original list is empty.
theorem reverse_eq_nil_imp_nil[T](items: List[T]) {
    reverse(items) = List.nil[T] implies items = List.nil[T]
} by {
    if reverse(items) = List.nil[T] {
        reverse_involution(items)
        reverse_nil[T]
        items = List.nil[T]
    }
}

/// If a list is empty, then its reverse is empty.
theorem reverse_nil_of_eq_nil[T](items: List[T]) {
    items = List.nil[T] implies reverse(items) = List.nil[T]
} by {
    if items = List.nil[T] {
        reverse_nil[T]
        reverse(items) = List.nil[T]
    }
}

/// Reversal is empty exactly when the original list is empty.
theorem reverse_eq_nil_iff[T](items: List[T]) {
    (reverse(items) = List.nil[T]) = (items = List.nil[T])
} by {
    reverse_eq_nil_imp_nil(items)
    reverse_nil_of_eq_nil(items)
    if reverse(items) = List.nil[T] {
        items = List.nil[T]
    }
    if items = List.nil[T] {
        reverse(items) = List.nil[T]
    }
}
