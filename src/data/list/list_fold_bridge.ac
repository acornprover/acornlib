from list import List, fold_left, fold_right
from algebra.monoid.monoid import Monoid

/// Folding right over a singleton applies the step function to the sole element and the initial value.
theorem fold_right_singleton[T, U](item: T, f: (T, U) -> U, init: U) {
    fold_right[T, U](List.singleton(item), f, init) = f(item, init)
} by {
    fold_right[T, U](List.cons(item, List.nil[T]), f, init) =
        f(item, fold_right[T, U](List.nil[T], f, init))
}

/// Folding left over a singleton applies the step function to the initial value and the sole element.
theorem fold_left_singleton[T, U](item: T, f: (U, T) -> U, init: U) {
    fold_left[T, U](List.singleton(item), f, init) = f(init, item)
} by {
}

/// The right-fold step for accumulating mapped list elements in a monoid.
define fold_right_monoid_step[T, A: Monoid](f: T -> A, item: T, acc: A) -> A {
    f(item) * acc
}

/// Right-fold a mapped list into a monoid using the monoid identity as the initial accumulator.
define fold_right_monoid[T, A: Monoid](items: List[T], f: T -> A) -> A {
    fold_right[T, A](items, fold_right_monoid_step[T, A](f), A.1)
}

/// The monoid right fold agrees with the underlying generic right fold.
theorem fold_right_monoid_eq_fold_right[T, A: Monoid](items: List[T], f: T -> A) {
    fold_right_monoid[T, A](items, f) =
        fold_right[T, A](items, fold_right_monoid_step[T, A](f), A.1)
}

/// The monoid right fold of the empty list is the monoid identity.
theorem fold_right_monoid_nil[T, A: Monoid](f: T -> A) {
    fold_right_monoid[T, A](List.nil[T], f) = A.1
} by {
}

/// Consing an element onto a list multiplies its mapped value on the left of the monoid right fold.
theorem fold_right_monoid_cons[T, A: Monoid](head: T, tail: List[T], f: T -> A) {
    fold_right_monoid[T, A](List.cons(head, tail), f) = f(head) * fold_right_monoid[T, A](tail, f)
} by {
    let step = fold_right_monoid_step[T, A](f)
    step(head, fold_right[T, A](tail, step, A.1)) =
        f(head) * fold_right[T, A](tail, step, A.1)
}

/// Right-folding a concatenation into a monoid factors as the product of the two right folds.
theorem fold_right_monoid_append[T, A: Monoid](left: List[T], right: List[T], f: T -> A) {
    fold_right_monoid[T, A](left + right, f) =
        fold_right_monoid[T, A](left, f) * fold_right_monoid[T, A](right, f)
} by {
    define p(items: List[T]) -> Bool {
        fold_right_monoid[T, A](items + right, f) =
            fold_right_monoid[T, A](items, f) * fold_right_monoid[T, A](right, f)
    }

    List.nil[T] + right = right
    fold_right_monoid_nil[T, A](f)
    fold_right_monoid[T, A](List.nil[T], f) = A.1
    A.1 * fold_right_monoid[T, A](right, f) = fold_right_monoid[T, A](right, f)
    p(List.nil[T])

    forall(head: T, tail: List[T]) {
        if p(tail) {
            fold_right_monoid_cons(head, tail + right, f)
            f(head) * fold_right_monoid[T, A](tail + right, f) =
                f(head) * (fold_right_monoid[T, A](tail, f) * fold_right_monoid[T, A](right, f))
            f(head) * (fold_right_monoid[T, A](tail, f) * fold_right_monoid[T, A](right, f)) =
                (f(head) * fold_right_monoid[T, A](tail, f)) * fold_right_monoid[T, A](right, f)
            fold_right_monoid_cons(head, tail, f)
            p(List.cons(head, tail))
        }
    }

    List.induction(function(items: List[T]) { p(items) })
    forall(items: List[T]) {
        p(items)
    }
}

/// The underlying generic right-fold append law for the monoid multiplication step.
theorem fold_right_monoid_append_fold_right[T, A: Monoid](left: List[T], right: List[T], f: T -> A) {
    fold_right[T, A](left + right, fold_right_monoid_step[T, A](f), A.1) =
        fold_right[T, A](left, fold_right_monoid_step[T, A](f), A.1) *
        fold_right[T, A](right, fold_right_monoid_step[T, A](f), A.1)
} by {
    fold_right_monoid_eq_fold_right(left + right, f)
    fold_right_monoid_eq_fold_right(left, f)
    fold_right_monoid_eq_fold_right(right, f)
    fold_right_monoid_append(left, right, f)
}
