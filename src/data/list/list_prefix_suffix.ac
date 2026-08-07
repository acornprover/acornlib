from list import List, add_contains_left, add_length, add_nil
from nat import Nat, alt_induction

/// The prefix containing the first `n` elements of a list, or the whole list if
/// `n` is longer than the list.
define list_take[T](items: List[T], n: Nat) -> List[T] {
    match n {
        Nat.zero {
            List.nil[T]
        }
        Nat.suc(pred) {
            match items {
                List.nil {
                    List.nil[T]
                }
                List.cons(head, tail) {
                    List.cons(head, list_take(tail, pred))
                }
            }
        }
    }
}

/// Taking zero elements yields the empty list.
theorem list_take_zero[T](items: List[T]) {
    list_take(items, Nat.0) = List.nil[T]
}

/// Taking any number of elements from the empty list yields the empty list.
theorem list_take_nil[T](n: Nat) {
    list_take(List.nil[T], n) = List.nil[T]
} by {
    define p(k: Nat) -> Bool {
        list_take(List.nil[T], k) = List.nil[T]
    }

    p(Nat.0)
    forall(k: Nat) {
        if p(k) {
            p(k.suc)
        }
    }
    alt_induction(p)
    forall(k: Nat) { p(k) }
}

/// Dropping any number of elements from the empty list yields the empty list.
theorem list_drop_nil[T](n: Nat) {
    List.nil[T].drop(n) = List.nil[T]
} by {
    define p(k: Nat) -> Bool {
        List.nil[T].drop(k) = List.nil[T]
    }

    p(Nat.0)
    forall(k: Nat) {
        if p(k) {
            p(k.suc)
        }
    }
    alt_induction(p)
    forall(k: Nat) { p(k) }
}

/// Taking a successor number of elements from a cons keeps the head and takes
/// the predecessor number of elements from the tail.
theorem list_take_cons_suc[T](head: T, tail: List[T], n: Nat) {
    list_take(List.cons(head, tail), n.suc) = List.cons(head, list_take(tail, n))
}

/// Dropping a successor number of elements from a cons drops the predecessor
/// number of elements from the tail.
theorem list_drop_cons_suc[T](head: T, tail: List[T], n: Nat) {
    List.cons(head, tail).drop(n.suc) = tail.drop(n)
} by {
    List.cons(head, tail).drop(n.suc) = List.cons(head, tail).tail.drop(n)
}

/// The prefix and the corresponding drop reassemble the original list.
theorem list_take_append_drop[T](items: List[T], n: Nat) {
    list_take(items, n) + items.drop(n) = items
} by {
    define p(xs: List[T]) -> Bool {
        forall(k: Nat) {
            list_take(xs, k) + xs.drop(k) = xs
        }
    }

    forall(k: Nat) {
        list_take_nil[T](k)
        list_drop_nil[T](k)
        list_take(List.nil[T], k) + List.nil[T].drop(k) = List.nil[T]
    }
    p(List.nil[T])

    forall(head: T, tail: List[T]) {
        if p(tail) {
            define q(k: Nat) -> Bool {
                list_take(List.cons(head, tail), k) + List.cons(head, tail).drop(k) =
                    List.cons(head, tail)
            }

            list_take_zero(List.cons(head, tail))
            list_take(List.cons(head, tail), Nat.0) = List.nil[T]
            List.cons(head, tail).drop(Nat.0) = List.cons(head, tail)
            List.nil[T] + List.cons(head, tail) = List.cons(head, tail)
            q(Nat.0)

            forall(k: Nat) {
                if q(k) {
                    list_take_cons_suc(head, tail, k)
                    list_drop_cons_suc(head, tail, k)

                    list_take(tail, k) + tail.drop(k) = tail
                    List.cons(head, list_take(tail, k)) + tail.drop(k) =
                        List.cons(head, list_take(tail, k) + tail.drop(k))
                    q(k.suc)
                }
            }
            alt_induction(q)
            forall(k: Nat) {
                q(k)
            }
            p(List.cons(head, tail))
        }
    }
    p(items)
}

/// Taking never increases length.
theorem list_take_length_at_most[T](items: List[T], n: Nat) {
    list_take(items, n).length <= items.length
} by {
    list_take_append_drop(items, n)
    add_length(list_take(items, n), items.drop(n))
}

/// Every element of a taken prefix is an element of the original list.
theorem list_take_contains_imp_contains[T](items: List[T], n: Nat, x: T) {
    list_take(items, n).contains(x) implies items.contains(x)
} by {
    if list_take(items, n).contains(x) {
        list_take_append_drop(items, n)
        add_contains_left(list_take(items, n), items.drop(n), x)
        items.contains(x)
    }
}

/// Dropping the full length of a list yields the empty list.
theorem list_drop_length_nil[T](items: List[T]) {
    items.drop(items.length) = List.nil[T]
} by {
    define p(xs: List[T]) -> Bool {
        xs.drop(xs.length) = List.nil[T]
    }

    p(List.nil[T])
    forall(head: T, tail: List[T]) {
        if p(tail) {
            List.cons(head, tail).drop(tail.length.suc) = tail.drop(tail.length)
            tail.drop(tail.length) = List.nil[T]
            p(List.cons(head, tail))
        }
    }
    forall(xs: List[T]) {
        p(xs)
    }
}

/// Taking the whole length of a list returns the list.
theorem list_take_length[T](items: List[T]) {
    list_take(items, items.length) = items
} by {
    list_take_append_drop(items, items.length)
    list_drop_length_nil(items)
    add_nil(list_take(items, items.length))
}
