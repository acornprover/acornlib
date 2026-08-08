from algebra.add_comm_monoid import AddCommMonoid
from list import List, length_range, map, map_add, partial, range_add_until,
    sum, sum_add
from data.list.list_prefix_suffix import list_drop_length_nil, list_take,
    list_take_append_drop, list_take_length
from nat import Nat

numerals Nat

/// Mapping over a concatenation and summing splits as the sum of the two mapped
/// parts.
theorem sum_map_add_split[T, A: AddCommMonoid](left: List[T], right: List[T], f: T -> A) {
    sum(map(left + right, f)) = sum(map(left, f)) + sum(map(right, f))
} by {
    map_add(left, right, f)
    sum_add(map(left, f), map(right, f))
}

/// The same mapped concatenation split with the two summands reversed.
theorem sum_map_add_split_comm[T, A: AddCommMonoid](left: List[T], right: List[T], f: T -> A) {
    sum(map(right, f)) + sum(map(left, f)) = sum(map(left + right, f))
} by {
    sum_map_add_split(left, right, f)
    sum(map(left, f)) + sum(map(right, f)) = sum(map(left + right, f))
    sum(map(right, f)) + sum(map(left, f)) = sum(map(left, f)) + sum(map(right, f))
}

/// The sum over a list splits into the sum over its taken prefix and the sum
/// over the corresponding dropped suffix.
theorem sum_map_take_drop[T, A: AddCommMonoid](items: List[T], n: Nat, f: T -> A) {
    sum(map(items, f)) = sum(map(list_take(items, n), f)) + sum(map(items.drop(n), f))
} by {
    list_take_append_drop(items, n)
    sum_map_add_split(list_take(items, n), items.drop(n), f)
}

/// The same take/drop split with the suffix sum first.
theorem sum_map_take_drop_comm[T, A: AddCommMonoid](items: List[T], n: Nat, f: T -> A) {
    sum(map(items.drop(n), f)) + sum(map(list_take(items, n), f)) = sum(map(items, f))
} by {
    sum_map_take_drop(items, n, f)
    sum(map(list_take(items, n), f)) + sum(map(items.drop(n), f)) = sum(map(items, f))
    sum(map(items.drop(n), f)) + sum(map(list_take(items, n), f)) =
        sum(map(list_take(items, n), f)) + sum(map(items.drop(n), f))
}

/// Taking the full length of a list preserves mapped sums.
theorem sum_map_take_length[T, A: AddCommMonoid](items: List[T], f: T -> A) {
    sum(map(list_take(items, items.length), f)) = sum(map(items, f))
} by {
    list_take_length(items)
}

/// Dropping the full length of a list leaves a zero mapped sum.
theorem sum_map_drop_length[T, A: AddCommMonoid](items: List[T], f: T -> A) {
    sum(map(items.drop(items.length), f)) = A.0
} by {
    list_drop_length_nil(items)
    map[T, A](List.nil[T], f) = List.nil[A]
    sum[A](List.nil[A]) = A.0
}

/// A partial sum splits at any earlier index into the previous partial sum and
/// the sum over the remaining interval.
theorem partial_interval_sum[A: AddCommMonoid](f: Nat -> A, a: Nat, b: Nat) {
    a <= b implies partial(f, b) = partial(f, a) + sum(map(a.until(b), f))
} by {
    if a <= b {
        range_add_until(a, b)
        sum_map_add_split(a.range, a.until(b), f)
        sum(map(a.range, f)) = partial(f, a)
        sum(map(b.range, f)) = partial(f, b)
        partial(f, b) = partial(f, a) + sum(map(a.until(b), f))
    }
}

/// The same interval split with the interval sum first.
theorem partial_interval_sum_comm[A: AddCommMonoid](f: Nat -> A, a: Nat, b: Nat) {
    a <= b implies sum(map(a.until(b), f)) + partial(f, a) = partial(f, b)
} by {
    if a <= b {
        partial_interval_sum(f, a, b)
        partial(f, a) + sum(map(a.until(b), f)) = partial(f, b)
        sum(map(a.until(b), f)) + partial(f, a) = partial(f, a) + sum(map(a.until(b), f))
        sum(map(a.until(b), f)) + partial(f, a) = partial(f, b)
    }
}

/// A partial sum with an additive offset splits into the initial block and the
/// following offset-length interval.
theorem partial_add_offset_sum[A: AddCommMonoid](f: Nat -> A, a: Nat, k: Nat) {
    partial(f, a + k) = partial(f, a) + sum(map(a.until(a + k), f))
} by {
    a <= a + k
    partial_interval_sum(f, a, a + k)
}

/// The same additive-offset interval split with the interval sum first.
theorem partial_add_offset_sum_comm[A: AddCommMonoid](f: Nat -> A, a: Nat, k: Nat) {
    sum(map(a.until(a + k), f)) + partial(f, a) = partial(f, a + k)
} by {
    a <= a + k
    partial_interval_sum_comm(f, a, a + k)
}

/// Splitting a partial sum at the predecessor index gives an interval form of
/// the successor step.
theorem partial_interval_suc[A: AddCommMonoid](f: Nat -> A, n: Nat) {
    partial(f, n.suc) = partial(f, n) + sum(map(n.until(n.suc), f))
} by {
    n <= n.suc
    partial_interval_sum(f, n, n.suc)
}

/// A partial sum over a range splits into sums over the taken prefix and the
/// dropped suffix of the range list.
theorem partial_range_take_drop_sum[A: AddCommMonoid](f: Nat -> A, n: Nat, k: Nat) {
    partial(f, n) = sum(map(list_take(n.range, k), f)) + sum(map(n.range.drop(k), f))
} by {
    sum_map_take_drop(n.range, k, f)
    sum(map(n.range, f)) = partial(f, n)
}

/// Taking all entries of a range list gives the corresponding partial sum.
theorem sum_map_take_range_length[A: AddCommMonoid](f: Nat -> A, n: Nat) {
    sum(map(list_take(n.range, n), f)) = partial(f, n)
} by {
    length_range(n)
    sum_map_take_length(n.range, f)
    sum(map(n.range, f)) = partial(f, n)
}

/// Dropping all entries of a range list gives a zero mapped sum.
theorem sum_map_drop_range_length[A: AddCommMonoid](f: Nat -> A, n: Nat) {
    sum(map(n.range.drop(n), f)) = A.0
} by {
    length_range(n)
    sum_map_drop_length(n.range, f)
}
