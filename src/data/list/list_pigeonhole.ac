from nat import Nat
from list import List, map
from data.list.list_unique_cons import unique_cons_head_fresh

numerals Nat

/// A collision in a mapped unique list occurs between two members of that list.
///
/// The strengthening of `pigeonhole_unique_map` that names where the colliding elements live.
/// Without it only a globally injective map can be shown to preserve uniqueness, which rules
/// out a map that is injective on the list but not everywhere — the situation whenever the map
/// is defined by a rule valid only on a range.
///
/// Kept in its own module rather than added to `src/list/list_sum.ac`: that file sits deep
/// enough in the library that changing it invalidates the certificates of everything
/// downstream, and nothing here needs to be visible that early.
theorem pigeonhole_unique_map_in[T, U](items: List[T], f: T -> U) {
    items.is_unique and not map(items, f).is_unique implies
    exists(x: T, y: T) {
        items.contains(x) and items.contains(y) and x != y and f(x) = f(y)
    }
} by {
    define p(l: List[T]) -> Bool {
        l.is_unique and not map(l, f).is_unique implies
        exists(a: T, b: T) {
            l.contains(a) and l.contains(b) and a != b and f(a) = f(b)
        }
    }
    map[T, U](List.nil[T], f) = List.nil[U]
    List.nil[U].unique = List.nil[U]
    List.nil[U].is_unique
    p(List.nil[T])
    forall(head: T, tail: List[T]) {
        if p(tail) {
            if List.cons(head, tail).is_unique and not map(List.cons(head, tail), f).is_unique {
                List.cons(head, tail).unique = List.cons(head, tail)
                tail.is_unique
                if not map(tail, f).is_unique {
                    exists(a: T, b: T) {
                        tail.contains(a) and tail.contains(b) and a != b and f(a) = f(b)
                    }
                    let (a: T, b: T) satisfy {
                        tail.contains(a) and tail.contains(b) and a != b and f(a) = f(b)
                    }
                    List.cons(head, tail).contains(a)
                    List.cons(head, tail).contains(b)
                    exists(u: T, v: T) {
                        List.cons(head, tail).contains(u) and List.cons(head, tail).contains(v)
                            and u != v and f(u) = f(v)
                    }
                }
                if map(tail, f).is_unique {
                    map(tail, f).unique = map(tail, f)
                    List.cons(f(head), map(tail, f)) = map(List.cons(head, tail), f)
                    map(List.cons(head, tail), f).unique != map(List.cons(head, tail), f)
                    map(tail, f).contains(f(head))
                    let x: T satisfy {
                        tail.contains(x) and f(x) = f(head)
                    }
                    tail.unique = tail
                    unique_cons_head_fresh(head, tail, x)
                    head != x
                    x != head
                    List.cons(head, tail).contains(x)
                    List.cons(head, tail).contains(head)
                    exists(u: T, v: T) {
                        List.cons(head, tail).contains(u) and List.cons(head, tail).contains(v)
                            and u != v and f(u) = f(v)
                    }
                }
                exists(u: T, v: T) {
                    List.cons(head, tail).contains(u) and List.cons(head, tail).contains(v)
                        and u != v and f(u) = f(v)
                }
            }
            p(List.cons(head, tail))
        }
    }
    p(items)
}

/// A map injective on the members of a unique list sends it to a unique list.
///
/// The local form of `injective_map_is_unique`. Injectivity is only needed where the list
/// actually goes, which is all a partially defined rule can supply.
theorem locally_injective_map_is_unique[T, U](items: List[T], f: T -> U) {
    items.is_unique and (forall(x: T, y: T) {
        items.contains(x) and items.contains(y) and f(x) = f(y) implies x = y
    }) implies map(items, f).is_unique
} by {
    if items.is_unique and forall(x: T, y: T) {
        items.contains(x) and items.contains(y) and f(x) = f(y) implies x = y
    } {
        if not map(items, f).is_unique {
            pigeonhole_unique_map_in(items, f)
            exists(x: T, y: T) {
                items.contains(x) and items.contains(y) and x != y and f(x) = f(y)
            }
            let (x: T, y: T) satisfy {
                items.contains(x) and items.contains(y) and x != y and f(x) = f(y)
            }
            (items.contains(x) and items.contains(y) and f(x) = f(y) implies x = y)
            x = y
            false
        }
        map(items, f).is_unique
    }
}
