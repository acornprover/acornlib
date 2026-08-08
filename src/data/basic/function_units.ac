/// Units for pointwise algebraic operations on functions.

from algebra.monoid.monoid import Monoid
from algebra.group import Group
from algebra.ring.ring import Ring
from algebra.units import Unit, is_monoid_unit, unit_val_is_monoid_unit, unit_inv_is_monoid_unit,
    monoid_unit_mul_eq_zero_left, monoid_unit_mul_eq_zero_right,
    monoid_unit_mul_cancel_left, monoid_unit_mul_cancel_right
from data.basic.function_algebra import pointwise_mul, pointwise_zero, pointwise_one, pointwise_inverse,
    pointwise_mul_one_right, pointwise_mul_one_left, pointwise_mul_assoc,
    pointwise_mul_inverse_right, pointwise_mul_inverse_left
from data.basic.functions import function_extensionality

/// True if a function has a two-sided pointwise inverse.
define is_pointwise_unit[T, M: Monoid](f: T -> M) -> Bool {
    exists(g: T -> M) {
        pointwise_mul(f, g) = pointwise_one[T, M] and
        pointwise_mul(g, f) = pointwise_one[T, M]
    }
}

/// The value function associated to a unit-valued function.
define unit_function_val[T, M: Monoid](f: T -> Unit[M], t: T) -> M {
    f(t).val
}

/// The inverse function associated to a unit-valued function.
define unit_function_inv[T, M: Monoid](f: T -> Unit[M], t: T) -> M {
    f(t).inv
}

/// The value function has the expected value at a point.
theorem unit_function_val_apply[T, M: Monoid](f: T -> Unit[M], t: T) {
    unit_function_val(f, t) = f(t).val
}

/// The inverse function has the expected value at a point.
theorem unit_function_inv_apply[T, M: Monoid](f: T -> Unit[M], t: T) {
    unit_function_inv(f, t) = f(t).inv
}

/// The value function multiplied by the inverse function is pointwise one.
theorem unit_function_val_mul_inv[T, M: Monoid](f: T -> Unit[M]) {
    pointwise_mul(unit_function_val(f), unit_function_inv(f)) = pointwise_one[T, M]
} by {
    forall(t: T) {
        f(t).val * f(t).inv = M.1
        pointwise_mul(unit_function_val(f), unit_function_inv(f), t) = pointwise_one[T, M](t)
    }
    function_extensionality(pointwise_mul(unit_function_val(f), unit_function_inv(f)), pointwise_one[T, M])
}

/// The inverse function multiplied by the value function is pointwise one.
theorem unit_function_inv_mul_val[T, M: Monoid](f: T -> Unit[M]) {
    pointwise_mul(unit_function_inv(f), unit_function_val(f)) = pointwise_one[T, M]
} by {
    forall(t: T) {
        f(t).inv * f(t).val = M.1
        pointwise_mul(unit_function_inv(f), unit_function_val(f), t) = pointwise_one[T, M](t)
    }
    function_extensionality(pointwise_mul(unit_function_inv(f), unit_function_val(f)), pointwise_one[T, M])
}

/// The value function associated to a unit-valued function is a pointwise unit.
theorem unit_function_val_is_pointwise_unit[T, M: Monoid](f: T -> Unit[M]) {
    is_pointwise_unit(unit_function_val(f))
} by {
}

/// The inverse function associated to a unit-valued function is a pointwise unit.
theorem unit_function_inv_is_pointwise_unit[T, M: Monoid](f: T -> Unit[M]) {
    is_pointwise_unit(unit_function_inv(f))
} by {
    unit_function_val_mul_inv(f)
    unit_function_inv_mul_val(f)
}

/// A pointwise unit has unit values.
theorem pointwise_unit_apply_is_monoid_unit[T, M: Monoid](f: T -> M, t: T) {
    is_pointwise_unit(f) implies is_monoid_unit(f(t))
} by {
    if is_pointwise_unit(f) {
        let g: T -> M satisfy {
            pointwise_mul(f, g) = pointwise_one[T, M] and
            pointwise_mul(g, f) = pointwise_one[T, M]
        }
        pointwise_one[T, M](t) = M.1
        g(t) * f(t) = M.1
        exists(a: M) {
            a = g(t) and f(t) * a = M.1 and a * f(t) = M.1
        }
    }
}

/// A stated two-sided pointwise inverse makes the original function a pointwise unit.
theorem pointwise_unit_of_inverse[T, M: Monoid](f: T -> M, g: T -> M) {
    pointwise_mul(f, g) = pointwise_one[T, M] and
    pointwise_mul(g, f) = pointwise_one[T, M]
    implies is_pointwise_unit(f)
} by {
    if pointwise_mul(f, g) = pointwise_one[T, M] and
        pointwise_mul(g, f) = pointwise_one[T, M] {
        exists(h: T -> M) {
            h = g and
            pointwise_mul(f, h) = pointwise_one[T, M] and
            pointwise_mul(h, f) = pointwise_one[T, M]
        }
    }
}

/// A stated two-sided pointwise inverse is itself a pointwise unit.
theorem pointwise_inverse_witness_is_pointwise_unit[T, M: Monoid](f: T -> M, g: T -> M) {
    pointwise_mul(f, g) = pointwise_one[T, M] and
    pointwise_mul(g, f) = pointwise_one[T, M]
    implies is_pointwise_unit(g)
} by {
    if pointwise_mul(f, g) = pointwise_one[T, M] and
        pointwise_mul(g, f) = pointwise_one[T, M] {
        exists(h: T -> M) {
            h = f and
            pointwise_mul(g, h) = pointwise_one[T, M] and
            pointwise_mul(h, g) = pointwise_one[T, M]
        }
    }
}

/// Pointwise-unit structure transports across equality of functions.
theorem pointwise_unit_of_eq[T, M: Monoid](f: T -> M, h: T -> M) {
    is_pointwise_unit(f) and f = h implies is_pointwise_unit(h)
} by {
    if is_pointwise_unit(f) and f = h {
        let g: T -> M satisfy {
            pointwise_mul(f, g) = pointwise_one[T, M] and
            pointwise_mul(g, f) = pointwise_one[T, M]
        }
        pointwise_mul(g, h) = pointwise_mul(g, f)
        pointwise_unit_of_inverse(h, g)
    }
}

/// Pointwise-unit structure transports across equality in the reverse orientation.
theorem pointwise_unit_of_eq_reverse[T, M: Monoid](f: T -> M, h: T -> M) {
    is_pointwise_unit(h) and f = h implies is_pointwise_unit(f)
} by {
    if is_pointwise_unit(h) and f = h {
        is_pointwise_unit(h) and h = f
        pointwise_unit_of_eq(h, f)
    }
}

/// The value of a unit-valued function is a monoid unit at each point.
theorem unit_function_val_apply_is_monoid_unit[T, M: Monoid](f: T -> Unit[M], t: T) {
    is_monoid_unit(unit_function_val(f, t))
} by {
    unit_val_is_monoid_unit(f(t))
}

/// The inverse of a unit-valued function is a monoid unit at each point.
theorem unit_function_inv_apply_is_monoid_unit[T, M: Monoid](f: T -> Unit[M], t: T) {
    is_monoid_unit(unit_function_inv(f, t))
} by {
    unit_inv_is_monoid_unit(f(t))
}

/// The pointwise one is a pointwise unit.
theorem pointwise_one_is_pointwise_unit[T, M: Monoid] {
    is_pointwise_unit(pointwise_one[T, M])
} by {
    pointwise_mul_one_right(pointwise_one[T, M])
}

/// Every function into a group is a pointwise unit.
theorem pointwise_unit_of_group[T, G: Group](f: T -> G) {
    is_pointwise_unit(f)
} by {
    pointwise_mul_inverse_right(f)
    pointwise_mul_inverse_left(f)
}

/// The pointwise product of two pointwise units is a pointwise unit.
theorem pointwise_mul_is_pointwise_unit[T, M: Monoid](f: T -> M, h: T -> M) {
    is_pointwise_unit(f) and is_pointwise_unit(h) implies
        is_pointwise_unit(pointwise_mul(f, h))
} by {
    if is_pointwise_unit(f) and is_pointwise_unit(h) {
        let fi: T -> M satisfy {
            pointwise_mul(f, fi) = pointwise_one[T, M] and
            pointwise_mul(fi, f) = pointwise_one[T, M]
        }
        let hi: T -> M satisfy {
            pointwise_mul(h, hi) = pointwise_one[T, M] and
            pointwise_mul(hi, h) = pointwise_one[T, M]
        }
        forall(t: T) {
            (f(t) * h(t)) * (hi(t) * fi(t)) = f(t) * (h(t) * (hi(t) * fi(t)))
            pointwise_one[T, M](t) = M.1
            (h(t) * hi(t)) * fi(t) = M.1 * fi(t)
            f(t) * (h(t) * (hi(t) * fi(t))) = f(t) * fi(t)
            pointwise_mul(pointwise_mul(f, h), pointwise_mul(hi, fi), t) = pointwise_one[T, M](t)
        }
        function_extensionality(pointwise_mul(pointwise_mul(f, h), pointwise_mul(hi, fi)), pointwise_one[T, M])
        forall(t: T) {
            (hi(t) * fi(t)) * (f(t) * h(t)) = hi(t) * (fi(t) * (f(t) * h(t)))
            pointwise_one[T, M](t) = M.1
            (fi(t) * f(t)) * h(t) = M.1 * h(t)
            hi(t) * (fi(t) * (f(t) * h(t))) = hi(t) * h(t)
            pointwise_mul(pointwise_mul(hi, fi), pointwise_mul(f, h), t) = pointwise_one[T, M](t)
        }
        function_extensionality(pointwise_mul(pointwise_mul(hi, fi), pointwise_mul(f, h)), pointwise_one[T, M])
        exists(g: T -> M) {
            g = pointwise_mul(hi, fi) and
            pointwise_mul(pointwise_mul(f, h), g) = pointwise_one[T, M] and
            pointwise_mul(g, pointwise_mul(f, h)) = pointwise_one[T, M]
        }
    }
}

/// Multiplication by a pointwise unit on the left has trivial zero kernel at each point.
theorem pointwise_unit_mul_eq_zero_left_apply[T, R: Ring](u: T -> R, f: T -> R, t: T) {
    is_pointwise_unit(u) and pointwise_mul(u, f) = pointwise_zero[T, R] implies
    f(t) = R.0
} by {
    if is_pointwise_unit(u) and pointwise_mul(u, f) = pointwise_zero[T, R] {
        pointwise_unit_apply_is_monoid_unit(u, t)
        pointwise_zero[T, R](t) = R.0
        u(t) * f(t) = R.0
        monoid_unit_mul_eq_zero_left(u(t), f(t))
        f(t) = R.0
    }
}

/// Multiplication by a pointwise unit on the left has trivial zero kernel.
theorem pointwise_unit_mul_eq_zero_left[T, R: Ring](u: T -> R, f: T -> R) {
    is_pointwise_unit(u) and pointwise_mul(u, f) = pointwise_zero[T, R] implies
    f = pointwise_zero[T, R]
} by {
    if is_pointwise_unit(u) and pointwise_mul(u, f) = pointwise_zero[T, R] {
        forall(t: T) {
            pointwise_unit_mul_eq_zero_left_apply(u, f, t)
            f(t) = pointwise_zero[T, R](t)
        }
        function_extensionality(f, pointwise_zero[T, R])
    }
}

/// Multiplication by a pointwise unit on the right has trivial zero kernel at each point.
theorem pointwise_unit_mul_eq_zero_right_apply[T, R: Ring](u: T -> R, f: T -> R, t: T) {
    is_pointwise_unit(u) and pointwise_mul(f, u) = pointwise_zero[T, R] implies
    f(t) = R.0
} by {
    if is_pointwise_unit(u) and pointwise_mul(f, u) = pointwise_zero[T, R] {
        pointwise_unit_apply_is_monoid_unit(u, t)
        pointwise_zero[T, R](t) = R.0
        f(t) * u(t) = R.0
        monoid_unit_mul_eq_zero_right(u(t), f(t))
        f(t) = R.0
    }
}

/// Multiplication by a pointwise unit on the right has trivial zero kernel.
theorem pointwise_unit_mul_eq_zero_right[T, R: Ring](u: T -> R, f: T -> R) {
    is_pointwise_unit(u) and pointwise_mul(f, u) = pointwise_zero[T, R] implies
    f = pointwise_zero[T, R]
} by {
    if is_pointwise_unit(u) and pointwise_mul(f, u) = pointwise_zero[T, R] {
        forall(t: T) {
            pointwise_unit_mul_eq_zero_right_apply(u, f, t)
            f(t) = pointwise_zero[T, R](t)
        }
        function_extensionality(f, pointwise_zero[T, R])
    }
}

/// Multiplication by a pointwise unit on the left is cancellative at each point.
theorem pointwise_unit_mul_cancel_left_apply[T, R: Ring](u: T -> R, f: T -> R, g: T -> R, t: T) {
    is_pointwise_unit(u) and pointwise_mul(u, f) = pointwise_mul(u, g) implies
    f(t) = g(t)
} by {
    if is_pointwise_unit(u) and pointwise_mul(u, f) = pointwise_mul(u, g) {
        pointwise_unit_apply_is_monoid_unit(u, t)
        u(t) * f(t) = u(t) * g(t)
        monoid_unit_mul_cancel_left(u(t), f(t), g(t))
        f(t) = g(t)
    }
}

/// Multiplication by a pointwise unit on the left is cancellative.
theorem pointwise_unit_mul_cancel_left[T, R: Ring](u: T -> R, f: T -> R, g: T -> R) {
    is_pointwise_unit(u) and pointwise_mul(u, f) = pointwise_mul(u, g) implies
    f = g
} by {
    if is_pointwise_unit(u) and pointwise_mul(u, f) = pointwise_mul(u, g) {
        forall(t: T) {
            pointwise_unit_mul_cancel_left_apply(u, f, g, t)
            f(t) = g(t)
        }
        function_extensionality(f, g)
    }
}

/// Multiplication by a pointwise unit on the right is cancellative at each point.
theorem pointwise_unit_mul_cancel_right_apply[T, R: Ring](u: T -> R, f: T -> R, g: T -> R, t: T) {
    is_pointwise_unit(u) and pointwise_mul(f, u) = pointwise_mul(g, u) implies
    f(t) = g(t)
} by {
    if is_pointwise_unit(u) and pointwise_mul(f, u) = pointwise_mul(g, u) {
        pointwise_unit_apply_is_monoid_unit(u, t)
        f(t) * u(t) = g(t) * u(t)
        monoid_unit_mul_cancel_right(u(t), f(t), g(t))
        f(t) = g(t)
    }
}

/// Multiplication by a pointwise unit on the right is cancellative.
theorem pointwise_unit_mul_cancel_right[T, R: Ring](u: T -> R, f: T -> R, g: T -> R) {
    is_pointwise_unit(u) and pointwise_mul(f, u) = pointwise_mul(g, u) implies
    f = g
} by {
    if is_pointwise_unit(u) and pointwise_mul(f, u) = pointwise_mul(g, u) {
        forall(t: T) {
            pointwise_unit_mul_cancel_right_apply(u, f, g, t)
            f(t) = g(t)
        }
        function_extensionality(f, g)
    }
}
