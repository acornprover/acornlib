/// Indicator functions for sets.

from data.basic.set import Set, empty_set_contains_eq, universal_set_contains_eq
from data.basic.function_algebra import pointwise_zero, pointwise_zero_apply
from data.basic.functions import function_extensionality
from algebra.zero import Zero

/// The indicator value of a set `s` with values in `A`.
define indicator[T, A: Zero](s: Set[T], f: T -> A, x: T) -> A {
    if s.contains(x) { f(x) } else { A.0 }
}

/// The indicator function associated to `s` and `f`.
define indicator_fn[T, A: Zero](s: Set[T], f: T -> A) -> T -> A {
    function(x: T) {
        indicator(s, f, x)
    }
}

/// The indicator function evaluates to the indicator value at each point.
theorem indicator_fn_apply[T, A: Zero](s: Set[T], f: T -> A, x: T) {
    indicator_fn(s, f)(x) = indicator(s, f, x)
}

/// The indicator value unfolds to its defining if-expression.
theorem indicator_apply[T, A: Zero](s: Set[T], f: T -> A, x: T) {
    indicator(s, f, x) = if s.contains(x) { f(x) } else { A.0 }
}

/// The indicator agrees with `f` on points of `s`.
theorem indicator_of_mem[T, A: Zero](s: Set[T], f: T -> A, x: T) {
    s.contains(x) implies indicator(s, f, x) = f(x)
} by {
    if s.contains(x) {
        indicator_apply(s, f, x)
        indicator(s, f, x) = if s.contains(x) { f(x) } else { A.0 }
        (if s.contains(x) { f(x) } else { A.0 }) = f(x)
        indicator(s, f, x) = f(x)
    }
}

/// The indicator is zero outside `s`.
theorem indicator_of_not_mem[T, A: Zero](s: Set[T], f: T -> A, x: T) {
    not s.contains(x) implies indicator(s, f, x) = A.0
} by {
    if not s.contains(x) {
        indicator_apply(s, f, x)
        indicator(s, f, x) = if s.contains(x) { f(x) } else { A.0 }
        (if s.contains(x) { f(x) } else { A.0 }) = A.0
        indicator(s, f, x) = A.0
    }
}

/// The indicator of the universal set is the original function.
theorem indicator_univ[T, A: Zero](f: T -> A) {
    indicator_fn(Set[T].universal_set, f) = f
} by {
    forall(x: T) {
        universal_set_contains_eq[T](x)
        indicator_of_mem(Set[T].universal_set, f, x)
        indicator_fn(Set[T].universal_set, f)(x) = f(x)
    }
    function_extensionality(indicator_fn(Set[T].universal_set, f), f)
}

/// The indicator of the empty set is the constant zero function.
theorem indicator_empty[T, A: Zero](f: T -> A) {
    indicator_fn(Set[T].empty_set, f) = constant[T, A](A.0)
} by {
    forall(x: T) {
        empty_set_contains_eq[T](x)
        indicator_of_not_mem(Set[T].empty_set, f, x)
        indicator_fn(Set[T].empty_set, f)(x) = A.0
        constant[T, A](A.0)(x) = A.0
        indicator_fn(Set[T].empty_set, f)(x) = constant[T, A](A.0)(x)
    }
    function_extensionality(indicator_fn(Set[T].empty_set, f), constant[T, A](A.0))
}

/// The indicator value of the pointwise zero function is zero.
theorem indicator_pointwise_zero_apply[T, A: Zero](s: Set[T], x: T) {
    indicator(s, pointwise_zero[T, A], x) = A.0
} by {
    if s.contains(x) {
        indicator_of_mem(s, pointwise_zero[T, A], x)
        indicator(s, pointwise_zero[T, A], x) = pointwise_zero[T, A](x)
        pointwise_zero_apply[T, A](x)
        pointwise_zero[T, A](x) = A.0
        indicator(s, pointwise_zero[T, A], x) = A.0
    } else {
        indicator_of_not_mem(s, pointwise_zero[T, A], x)
        indicator(s, pointwise_zero[T, A], x) = A.0
    }
}

/// The indicator function of the pointwise zero function is pointwise zero.
theorem indicator_pointwise_zero[T, A: Zero](s: Set[T]) {
    indicator_fn(s, pointwise_zero[T, A]) = pointwise_zero[T, A]
} by {
    forall(x: T) {
        indicator_pointwise_zero_apply[T, A](s, x)
        indicator_fn(s, pointwise_zero[T, A])(x) = A.0
        pointwise_zero_apply[T, A](x)
        pointwise_zero[T, A](x) = A.0
        indicator_fn(s, pointwise_zero[T, A])(x) = pointwise_zero[T, A](x)
    }
    function_extensionality(indicator_fn(s, pointwise_zero[T, A]), pointwise_zero[T, A])
}

/// The indicator of the empty set is the pointwise zero function.
theorem indicator_empty_pointwise_zero[T, A: Zero](f: T -> A) {
    indicator_fn(Set[T].empty_set, f) = pointwise_zero[T, A]
} by {
    forall(x: T) {
        empty_set_contains_eq[T](x)
        indicator_of_not_mem(Set[T].empty_set, f, x)
        indicator_fn(Set[T].empty_set, f)(x) = A.0
        pointwise_zero_apply[T, A](x)
        pointwise_zero[T, A](x) = A.0
        indicator_fn(Set[T].empty_set, f)(x) = pointwise_zero[T, A](x)
    }
    function_extensionality(indicator_fn(Set[T].empty_set, f), pointwise_zero[T, A])
}
