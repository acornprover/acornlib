from data.basic.set import Set, set_image, set_preimage, set_preimage_image_of_injective,
    set_image_preimage_of_surjective, set_image_monotone, set_preimage_monotone,
    set_image_subset_imp_subset_of_injective, set_preimage_subset_imp_subset_of_surjective,
    set_image_eq_imp_eq_of_injective, set_preimage_eq_imp_eq_of_surjective,
    set_image_intersection_of_injective,
    set_image_difference_of_injective, set_image_compl_of_bijection,
    set_preimage_compl, set_ext, difference_contains_eq, set_preimage_contains_eq
from data.basic.functions import is_bijection_fn, is_injective_fn, is_surjective_fn,
    bijection_fn_is_injective, bijection_fn_is_surjective

/// A bijection makes preimage after image equal to the original source set.
theorem set_preimage_image_of_bijection[T, U](s: Set[T], f: T -> U) {
    is_bijection_fn(f) implies set_preimage(f, set_image(s, f)) = s
} by {
    if is_bijection_fn(f) {
        bijection_fn_is_injective(f)
        is_injective_fn(f)
        set_preimage_image_of_injective(s, f)
        set_preimage(f, set_image(s, f)) = s
    }
}

/// A bijection makes image after preimage equal to the original target set.
theorem set_image_preimage_of_bijection[T, U](s: Set[U], f: T -> U) {
    is_bijection_fn(f) implies set_image(set_preimage(f, s), f) = s
} by {
    if is_bijection_fn(f) {
        bijection_fn_is_surjective(f)
        is_surjective_fn(f)
        set_image_preimage_of_surjective(s, f)
        set_image(set_preimage(f, s), f) = s
    }
}

/// Under a bijection, source-set inclusion is equivalent to inclusion of images.
theorem subset_iff_set_image_subset_of_bijection[T, U](a: Set[T], b: Set[T], f: T -> U) {
    is_bijection_fn(f) implies (a.subset(b) = set_image(a, f).subset(set_image(b, f)))
} by {
    if is_bijection_fn(f) {
        if a.subset(b) {
            set_image_monotone(a, b, f)
            set_image(a, f).subset(set_image(b, f))
        }
        if set_image(a, f).subset(set_image(b, f)) {
            bijection_fn_is_injective(f)
            is_injective_fn(f)
            set_image_subset_imp_subset_of_injective(a, b, f)
            a.subset(b)
        }
        a.subset(b) = set_image(a, f).subset(set_image(b, f))
    }
}

/// Under a bijection, target-set inclusion is equivalent to inclusion of preimages.
theorem subset_iff_set_preimage_subset_of_bijection[T, U](a: Set[U], b: Set[U], f: T -> U) {
    is_bijection_fn(f) implies (a.subset(b) = set_preimage(f, a).subset(set_preimage(f, b)))
} by {
    if is_bijection_fn(f) {
        if a.subset(b) {
            set_preimage_monotone(f, a, b)
            set_preimage(f, a).subset(set_preimage(f, b))
        }
        if set_preimage(f, a).subset(set_preimage(f, b)) {
            bijection_fn_is_surjective(f)
            is_surjective_fn(f)
            set_preimage_subset_imp_subset_of_surjective(a, b, f)
            a.subset(b)
        }
        a.subset(b) = set_preimage(f, a).subset(set_preimage(f, b))
    }
}

/// Under a bijection, a target set lies in the image of a source set exactly when its preimage lies in that source set.
theorem set_preimage_subset_iff_subset_set_image_of_bijection[T, U](a: Set[T], b: Set[U], f: T -> U) {
    is_bijection_fn(f) implies (set_preimage(f, b).subset(a) = b.subset(set_image(a, f)))
} by {
    if is_bijection_fn(f) {
        if set_preimage(f, b).subset(a) {
            set_image_monotone(set_preimage(f, b), a, f)
            set_image(set_preimage(f, b), f).subset(set_image(a, f))
            set_image_preimage_of_bijection(b, f)
            set_image(set_preimage(f, b), f) = b
            b.subset(set_image(a, f))
        }
        if b.subset(set_image(a, f)) {
            set_preimage_monotone(f, b, set_image(a, f))
            set_preimage(f, b).subset(set_preimage(f, set_image(a, f)))
            set_preimage_image_of_bijection(a, f)
            set_preimage(f, set_image(a, f)) = a
            set_preimage(f, b).subset(a)
        }
        set_preimage(f, b).subset(a) = b.subset(set_image(a, f))
    }
}

/// Under a bijection, saying that the image is a target set is equivalent to saying that the source is its preimage.
theorem set_image_eq_iff_eq_preimage_of_bijection[T, U](a: Set[T], b: Set[U], f: T -> U) {
    is_bijection_fn(f) implies ((set_image(a, f) = b) = (a = set_preimage(f, b)))
} by {
    if is_bijection_fn(f) {
        if set_image(a, f) = b {
            set_preimage(f, set_image(a, f)) = set_preimage(f, b)
            set_preimage_image_of_bijection(a, f)
            set_preimage(f, set_image(a, f)) = a
            a = set_preimage(f, b)
        }
        if a = set_preimage(f, b) {
            set_image(a, f) = set_image(set_preimage(f, b), f)
            set_image_preimage_of_bijection(b, f)
            set_image(set_preimage(f, b), f) = b
            set_image(a, f) = b
        }
        (set_image(a, f) = b) = (a = set_preimage(f, b))
    }
}

/// Under a bijection, source sets are equal exactly when their images are equal.
theorem set_image_eq_iff_eq_of_bijection[T, U](a: Set[T], b: Set[T], f: T -> U) {
    is_bijection_fn(f) implies ((set_image(a, f) = set_image(b, f)) = (a = b))
} by {
    if is_bijection_fn(f) {
        if set_image(a, f) = set_image(b, f) {
            bijection_fn_is_injective(f)
            is_injective_fn(f)
            set_image_eq_imp_eq_of_injective(a, b, f)
            a = b
        }
        if a = b {
            set_image(a, f) = set_image(b, f)
        }
        (set_image(a, f) = set_image(b, f)) = (a = b)
    }
}

/// Under a bijection, target sets are equal exactly when their preimages are equal.
theorem set_preimage_eq_iff_eq_of_bijection[T, U](a: Set[U], b: Set[U], f: T -> U) {
    is_bijection_fn(f) implies ((set_preimage(f, a) = set_preimage(f, b)) = (a = b))
} by {
    if is_bijection_fn(f) {
        if set_preimage(f, a) = set_preimage(f, b) {
            bijection_fn_is_surjective(f)
            is_surjective_fn(f)
            set_preimage_eq_imp_eq_of_surjective(a, b, f)
            a = b
        }
        if a = b {
            set_preimage(f, a) = set_preimage(f, b)
        }
        (set_preimage(f, a) = set_preimage(f, b)) = (a = b)
    }
}

/// A bijective image preserves binary intersections exactly.
theorem set_image_intersection_of_bijection[T, U](a: Set[T], b: Set[T], f: T -> U) {
    is_bijection_fn(f) implies
    set_image(a.intersection(b), f) = set_image(a, f).intersection(set_image(b, f))
} by {
    if is_bijection_fn(f) {
        bijection_fn_is_injective(f)
        is_injective_fn(f)
        set_image_intersection_of_injective(a, b, f)
        set_image(a.intersection(b), f) = set_image(a, f).intersection(set_image(b, f))
    }
}

/// A bijective image preserves set differences exactly.
theorem set_image_difference_of_bijection[T, U](a: Set[T], b: Set[T], f: T -> U) {
    is_bijection_fn(f) implies
    set_image(a.difference(b), f) = set_image(a, f).difference(set_image(b, f))
} by {
    if is_bijection_fn(f) {
        bijection_fn_is_injective(f)
        is_injective_fn(f)
        set_image_difference_of_injective(a, b, f)
        set_image(a.difference(b), f) = set_image(a, f).difference(set_image(b, f))
    }
}

/// Under a bijection, the preimage of the complement of an image is the source complement.
theorem set_preimage_image_complement_of_bijection[T, U](s: Set[T], f: T -> U) {
    is_bijection_fn(f) implies set_preimage(f, set_image(s, f).c) = s.c
} by {
    if is_bijection_fn(f) {
        set_preimage_compl(f, set_image(s, f))
        set_preimage(f, set_image(s, f).c) = set_preimage(f, set_image(s, f)).c
        set_preimage_image_of_bijection(s, f)
        set_preimage(f, set_image(s, f)) = s
        set_preimage(f, set_image(s, f)).c = s.c
        set_preimage(f, set_image(s, f).c) = s.c
    }
}

/// Under a bijection, the image of the complement of a preimage is the target complement.
theorem set_image_preimage_complement_of_bijection[T, U](s: Set[U], f: T -> U) {
    is_bijection_fn(f) implies set_image(set_preimage(f, s).c, f) = s.c
} by {
    if is_bijection_fn(f) {
        set_image_compl_of_bijection(set_preimage(f, s), f)
        set_image(set_preimage(f, s).c, f) = set_image(set_preimage(f, s), f).c
        set_image_preimage_of_bijection(s, f)
        set_image(set_preimage(f, s), f) = s
        set_image(set_preimage(f, s), f).c = s.c
        set_image(set_preimage(f, s).c, f) = s.c
    }
}

/// Preimage preserves set differences.
theorem set_preimage_difference[T, U](f: T -> U, a: Set[U], b: Set[U]) {
    set_preimage(f, a.difference(b)) = set_preimage(f, a).difference(set_preimage(f, b))
} by {
    let u = set_preimage(f, a.difference(b))
    let v = set_preimage(f, a).difference(set_preimage(f, b))
    forall(x: T) {
        if u.contains(x) {
            set_preimage_contains_eq(f, a.difference(b), x)
            difference_contains_eq(a, b, f(x))
            a.contains(f(x))
            not b.contains(f(x))
            set_preimage_contains_eq(f, a, x)
            set_preimage(f, a).contains(x)
            set_preimage_contains_eq(f, b, x)
            not set_preimage(f, b).contains(x)
            difference_contains_eq(set_preimage(f, a), set_preimage(f, b), x)
            v.contains(x)
        }
        if v.contains(x) {
            difference_contains_eq(set_preimage(f, a), set_preimage(f, b), x)
            let pre_a = set_preimage(f, a)
            let pre_b = set_preimage(f, b)
            pre_a.contains(x)
            not pre_b.contains(x)
            set_preimage_contains_eq(f, a, x)
            a.contains(f(x))
            set_preimage_contains_eq(f, b, x)
            not b.contains(f(x))
            difference_contains_eq(a, b, f(x))
            a.difference(b).contains(f(x))
            set_preimage_contains_eq(f, a.difference(b), x)
            u.contains(x)
        }
        u.contains(x) = v.contains(x)
    }
    set_ext(u, v)
}
