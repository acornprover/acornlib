/// Componentwise order on pairs.

from order import PartialOrder, lte_refl, lt_imp_lte, lt_imp_ne, lt_of_lte_of_ne
from lte import LTE
from order_iso import is_order_iso_pair, OrderIso, order_iso_pair_left_inverse, order_iso_pair_right_inverse,
    order_iso_pair_map_is_order_embedding, order_iso_pair_inv_is_order_embedding,
    order_iso_new_map, order_iso_new_inv, order_iso_ext,
    identity_order_iso, identity_order_iso_map, identity_order_iso_inv,
    inverse_order_iso, inverse_order_iso_map, inverse_order_iso_inv,
    compose_order_iso, compose_order_iso_map, compose_order_iso_inv,
    order_iso_apply, order_iso_unapply, order_iso_apply_at, order_iso_unapply_at
from order_relation import is_partial_order_relation
from order import is_monotone, is_antitone, is_order_embedding, monotone_step, antitone_step,
    order_embedding_apply_le, order_embedding_reflects_lte, order_embedding_is_monotone
from pair import Pair, pair_ext, pair_new_first, pair_new_second, pair_map, pair_map_first,
    pair_map_second, swap_swap, swap_first, swap_second, pair_map_comp, pair_map_identity
from data.basic.functions import compose, identity_fn, function_extensionality
from data.basic.relation_basic import is_reflexive, is_transitive, is_antisymmetric, is_irreflexive, is_asymmetric,
    transitive_step, antisymmetric_eq

/// The componentwise non-strict order on pairs.
define pair_lte[A: PartialOrder, B: PartialOrder](p: Pair[A, B], q: Pair[A, B]) -> Bool {
    p.first <= q.first and p.second <= q.second
}

/// The strict order associated to the componentwise non-strict order on pairs.
define pair_lt[A: PartialOrder, B: PartialOrder](p: Pair[A, B], q: Pair[A, B]) -> Bool {
    pair_lte(p, q) and p != q
}

/// Pair order is exactly componentwise order.
theorem pair_lte_eq[A: PartialOrder, B: PartialOrder](p: Pair[A, B], q: Pair[A, B]) {
    pair_lte(p, q) = (p.first <= q.first and p.second <= q.second)
}

/// Strict pair order is non-strict pair order together with disequality.
theorem pair_lt_eq[A: PartialOrder, B: PartialOrder](p: Pair[A, B], q: Pair[A, B]) {
    pair_lt(p, q) = (pair_lte(p, q) and p != q)
}

/// Componentwise inequalities give an inequality of pairs.
theorem pair_lte_of_components[A: PartialOrder, B: PartialOrder](p: Pair[A, B], q: Pair[A, B]) {
    p.first <= q.first and p.second <= q.second implies pair_lte(p, q)
} by {
    if p.first <= q.first and p.second <= q.second {
        pair_lte_eq(p, q)
        pair_lte(p, q)
    }
}

/// An inequality of pairs gives an inequality of first components.
theorem pair_lte_first[A: PartialOrder, B: PartialOrder](p: Pair[A, B], q: Pair[A, B]) {
    pair_lte(p, q) implies p.first <= q.first
} by {
    if pair_lte(p, q) {
        pair_lte_eq(p, q)
        p.first <= q.first
    }
}

/// An inequality of pairs gives an inequality of second components.
theorem pair_lte_second[A: PartialOrder, B: PartialOrder](p: Pair[A, B], q: Pair[A, B]) {
    pair_lte(p, q) implies p.second <= q.second
} by {
    if pair_lte(p, q) {
        pair_lte_eq(p, q)
        p.second <= q.second
    }
}

/// The componentwise order on pairs is reflexive.
theorem pair_lte_refl[A: PartialOrder, B: PartialOrder](p: Pair[A, B]) {
    pair_lte(p, p)
} by {
    lte_refl(p.first)
    lte_refl(p.second)
    pair_lte_of_components(p, p)
}

/// The componentwise order on pairs is transitive.
theorem pair_lte_trans[A: PartialOrder, B: PartialOrder](p: Pair[A, B], q: Pair[A, B], r: Pair[A, B]) {
    pair_lte(p, q) and pair_lte(q, r) implies pair_lte(p, r)
} by {
    if pair_lte(p, q) and pair_lte(q, r) {
        pair_lte_first(p, q)
        pair_lte_first(q, r)
        if p.first <= q.first and q.first <= r.first {
            transitive_step(A.lte, p.first, q.first, r.first)
            p.first <= r.first
        }
        pair_lte_second(p, q)
        pair_lte_second(q, r)
        if p.second <= q.second and q.second <= r.second {
            transitive_step(B.lte, p.second, q.second, r.second)
            p.second <= r.second
        }
        pair_lte_of_components(p, r)
        pair_lte(p, r)
    }
}

/// The componentwise order on pairs is antisymmetric.
theorem pair_lte_antisymm[A: PartialOrder, B: PartialOrder](p: Pair[A, B], q: Pair[A, B]) {
    pair_lte(p, q) and pair_lte(q, p) implies p = q
} by {
    if pair_lte(p, q) and pair_lte(q, p) {
        pair_lte_first(p, q)
        pair_lte_first(q, p)
        if p.first <= q.first and q.first <= p.first {
            antisymmetric_eq(A.lte, p.first, q.first)
            p.first = q.first
        }
        pair_lte_second(p, q)
        pair_lte_second(q, p)
        if p.second <= q.second and q.second <= p.second {
            antisymmetric_eq(B.lte, p.second, q.second)
            p.second = q.second
        }
        pair_ext(p, q)
        p = q
    }
}

/// Strict pair order implies non-strict pair order.
theorem pair_lt_imp_lte[A: PartialOrder, B: PartialOrder](p: Pair[A, B], q: Pair[A, B]) {
    pair_lt(p, q) implies pair_lte(p, q)
} by {
    if pair_lt(p, q) {
        pair_lt_eq(p, q)
        pair_lte(p, q)
    }
}

/// Strict pair order implies disequality.
theorem pair_lt_imp_ne[A: PartialOrder, B: PartialOrder](p: Pair[A, B], q: Pair[A, B]) {
    pair_lt(p, q) implies p != q
} by {
    if pair_lt(p, q) {
        pair_lt_eq(p, q)
        p != q
    }
}

/// Non-strict pair order plus disequality gives strict pair order.
theorem pair_lt_of_lte_of_ne[A: PartialOrder, B: PartialOrder](p: Pair[A, B], q: Pair[A, B]) {
    pair_lte(p, q) and p != q implies pair_lt(p, q)
} by {
    if pair_lte(p, q) and p != q {
        pair_lt_eq(p, q)
        pair_lt(p, q)
    }
}

/// Strict pair order is exactly non-strict pair order plus disequality.
theorem pair_lt_iff_lte_and_ne[A: PartialOrder, B: PartialOrder](p: Pair[A, B], q: Pair[A, B]) {
    pair_lt(p, q) = (pair_lte(p, q) and p != q)
} by {
    pair_lt_eq(p, q)
}

/// No pair is strictly below itself.
theorem pair_not_lt_self[A: PartialOrder, B: PartialOrder](p: Pair[A, B]) {
    not pair_lt(p, p)
} by {
    if pair_lt(p, p) {
        pair_lt_imp_ne(p, p)
        p != p
        false
    }
}

/// Strict pair order is irreflexive.
theorem pair_lt_irrefl[A: PartialOrder, B: PartialOrder](p: Pair[A, B]) {
    not pair_lt(p, p)
} by {
    pair_not_lt_self(p)
}

/// Strict pair order is transitive.
theorem pair_lt_trans[A: PartialOrder, B: PartialOrder](p: Pair[A, B], q: Pair[A, B], r: Pair[A, B]) {
    pair_lt(p, q) and pair_lt(q, r) implies pair_lt(p, r)
} by {
    if pair_lt(p, q) and pair_lt(q, r) {
        pair_lt_imp_lte(p, q)
        pair_lte(p, q)
        pair_lt_imp_lte(q, r)
        pair_lte(q, r)
        pair_lte_trans(p, q, r)
        pair_lte(p, r)
        if p = r {
            pair_lte(q, r)
            pair_lte(p, q)
            pair_lte(r, q)
            pair_lte_antisymm(q, r)
            q = r
            pair_lt_imp_ne(q, r)
            q != r
            false
        }
        p != r
        pair_lt_of_lte_of_ne(p, r)
        pair_lt(p, r)
    }
}

/// Strict pair order is asymmetric.
theorem pair_lt_asymm[A: PartialOrder, B: PartialOrder](p: Pair[A, B], q: Pair[A, B]) {
    pair_lt(p, q) implies not pair_lt(q, p)
} by {
    if pair_lt(p, q) {
        if pair_lt(q, p) {
            pair_lt_trans(p, q, p)
            pair_lt(p, p)
            pair_not_lt_self(p)
            false
        }
    }
}

/// A strict pair bound followed by a non-strict pair bound is strict.
theorem pair_lt_of_lt_of_lte[A: PartialOrder, B: PartialOrder](p: Pair[A, B], q: Pair[A, B], r: Pair[A, B]) {
    pair_lt(p, q) and pair_lte(q, r) implies pair_lt(p, r)
} by {
    if pair_lt(p, q) and pair_lte(q, r) {
        pair_lt_imp_lte(p, q)
        pair_lte(p, q)
        pair_lte_trans(p, q, r)
        pair_lte(p, r)
        if p = r {
            pair_lte(q, r)
            pair_lte(p, q)
            pair_lte(r, q)
            pair_lte_antisymm(q, r)
            q = r
            pair_lt_imp_ne(p, q)
            p != q
            false
        }
        p != r
        pair_lt_of_lte_of_ne(p, r)
        pair_lt(p, r)
    }
}

/// A non-strict pair bound followed by a strict pair bound is strict.
theorem pair_lt_of_lte_of_lt[A: PartialOrder, B: PartialOrder](p: Pair[A, B], q: Pair[A, B], r: Pair[A, B]) {
    pair_lte(p, q) and pair_lt(q, r) implies pair_lt(p, r)
} by {
    if pair_lte(p, q) and pair_lt(q, r) {
        pair_lt_imp_lte(q, r)
        pair_lte(q, r)
        pair_lte_trans(p, q, r)
        pair_lte(p, r)
        if p = r {
            pair_lte(q, r)
            pair_lte(p, q)
            pair_lte(q, p)
            pair_lte_antisymm(p, q)
            p = q
            pair_lt_imp_ne(q, r)
            q != r
            false
        }
        p != r
        pair_lt_of_lte_of_ne(p, r)
        pair_lt(p, r)
    }
}

/// A strict first-component comparison gives strict pair order when the second component is non-strict.
theorem pair_lt_of_first_lt_of_second_lte[A: PartialOrder, B: PartialOrder](p: Pair[A, B], q: Pair[A, B]) {
    p.first < q.first and p.second <= q.second implies pair_lt(p, q)
} by {
    if p.first < q.first and p.second <= q.second {
        lt_imp_lte(p.first, q.first)
        p.first <= q.first
        pair_lte_of_components(p, q)
        pair_lte(p, q)
        if p = q {
            p.first = q.first
            lt_imp_ne(p.first, q.first)
            p.first != q.first
            false
        }
        p != q
        pair_lt_of_lte_of_ne(p, q)
        pair_lt(p, q)
    }
}

/// A strict second-component comparison gives strict pair order when the first component is non-strict.
theorem pair_lt_of_first_lte_of_second_lt[A: PartialOrder, B: PartialOrder](p: Pair[A, B], q: Pair[A, B]) {
    p.first <= q.first and p.second < q.second implies pair_lt(p, q)
} by {
    if p.first <= q.first and p.second < q.second {
        lt_imp_lte(p.second, q.second)
        p.second <= q.second
        pair_lte_of_components(p, q)
        pair_lte(p, q)
        if p = q {
            p.second = q.second
            lt_imp_ne(p.second, q.second)
            p.second != q.second
            false
        }
        p != q
        pair_lt_of_lte_of_ne(p, q)
        pair_lt(p, q)
    }
}

/// The componentwise order on pairs is reflexive as a relation.
theorem pair_lte_is_reflexive[A: PartialOrder, B: PartialOrder] {
    is_reflexive(pair_lte[A, B])
} by {
    forall(p: Pair[A, B]) {
        pair_lte_refl(p)
    }
}

/// The componentwise order on pairs is transitive as a relation.
theorem pair_lte_is_transitive[A: PartialOrder, B: PartialOrder] {
    is_transitive(pair_lte[A, B])
} by {
    is_transitive(pair_lte[A, B]) = forall(p: Pair[A, B], q: Pair[A, B], r: Pair[A, B]) {
        pair_lte(p, q) and pair_lte(q, r) implies pair_lte(p, r)
    }
    forall(p: Pair[A, B], q: Pair[A, B], r: Pair[A, B]) {
        if pair_lte(p, q) and pair_lte(q, r) {
            pair_lte_trans(p, q, r)
            pair_lte(p, r)
        }
    }
}

/// The componentwise order on pairs is antisymmetric as a relation.
theorem pair_lte_is_antisymmetric[A: PartialOrder, B: PartialOrder] {
    is_antisymmetric(pair_lte[A, B])
} by {
    forall(p: Pair[A, B], q: Pair[A, B]) {
        if pair_lte(p, q) and pair_lte(q, p) {
            pair_lte_antisymm(p, q)
            p = q
        }
    }
}

/// The componentwise order on pairs has the data of a partial order.
theorem pair_lte_is_partial_order_relation[A: PartialOrder, B: PartialOrder] {
    is_partial_order_relation(pair_lte[A, B])
} by {
    pair_lte_is_reflexive[A, B]
    pair_lte_is_transitive[A, B]
    pair_lte_is_antisymmetric[A, B]
    is_partial_order_relation(pair_lte[A, B])
}

/// The strict componentwise order is irreflexive as a relation.
theorem pair_lt_is_irreflexive[A: PartialOrder, B: PartialOrder] {
    is_irreflexive(pair_lt[A, B])
} by {
    forall(p: Pair[A, B]) {
        pair_not_lt_self(p)
    }
}

/// The strict componentwise order is transitive as a relation.
theorem pair_lt_is_transitive[A: PartialOrder, B: PartialOrder] {
    is_transitive(pair_lt[A, B])
} by {
    is_transitive(pair_lt[A, B]) = forall(p: Pair[A, B], q: Pair[A, B], r: Pair[A, B]) {
        pair_lt(p, q) and pair_lt(q, r) implies pair_lt(p, r)
    }
    forall(p: Pair[A, B], q: Pair[A, B], r: Pair[A, B]) {
        if pair_lt(p, q) and pair_lt(q, r) {
            pair_lt_trans(p, q, r)
            pair_lt(p, r)
        }
    }
}

/// The strict componentwise order is asymmetric as a relation.
theorem pair_lt_is_asymmetric[A: PartialOrder, B: PartialOrder] {
    is_asymmetric(pair_lt[A, B])
} by {
    forall(p: Pair[A, B], q: Pair[A, B]) {
        if pair_lt(p, q) {
            pair_lt_asymm(p, q)
            not pair_lt(q, p)
        }
    }
}

/// A new pair is below another new pair exactly when both components are below.
theorem pair_new_lte_iff[A: PartialOrder, B: PartialOrder](a: A, b: B, c: A, d: B) {
    pair_lte(Pair.new(a, b), Pair.new(c, d)) = (a <= c and b <= d)
} by {
    let p = Pair.new(a, b)
    let q = Pair.new(c, d)
    pair_new_first(a, b)
    pair_new_second(a, b)
    pair_new_first(c, d)
    pair_new_second(c, d)
    pair_lte_eq(p, q)
    pair_lte(p, q) = (a <= c and b <= d)
}

/// A new pair is strictly below another new pair exactly when it is componentwise below and distinct.
theorem pair_new_lt_iff[A: PartialOrder, B: PartialOrder](a: A, b: B, c: A, d: B) {
    pair_lt(Pair.new(a, b), Pair.new(c, d)) =
    (a <= c and b <= d and Pair.new(a, b) != Pair.new(c, d))
} by {
    let p = Pair.new(a, b)
    let q = Pair.new(c, d)
    pair_new_lte_iff(a, b, c, d)
    pair_lt_eq(p, q)
    pair_lt(p, q) = (a <= c and b <= d and p != q)
}

/// A new pair is below a pair exactly when both components are below the projections.
theorem pair_new_lte_iff_right[A: PartialOrder, B: PartialOrder](a: A, b: B, q: Pair[A, B]) {
    pair_lte(Pair.new(a, b), q) = (a <= q.first and b <= q.second)
} by {
    let p = Pair.new(a, b)
    pair_new_first(a, b)
    pair_new_second(a, b)
    pair_lte_eq(p, q)
    pair_lte(p, q) = (a <= q.first and b <= q.second)
}

/// A pair is below a new pair exactly when both projections are below the components.
theorem pair_lte_new_iff_left[A: PartialOrder, B: PartialOrder](p: Pair[A, B], a: A, b: B) {
    pair_lte(p, Pair.new(a, b)) = (p.first <= a and p.second <= b)
} by {
    let q = Pair.new(a, b)
    pair_new_first(a, b)
    pair_new_second(a, b)
    pair_lte_eq(p, q)
    pair_lte(p, q) = (p.first <= a and p.second <= b)
}

/// The first projection is monotone for the componentwise order.
theorem pair_first_monotone_for_lte[A: PartialOrder, B: PartialOrder](p: Pair[A, B], q: Pair[A, B]) {
    pair_lte(p, q) implies p.first <= q.first
} by {
    if pair_lte(p, q) {
        pair_lte_first(p, q)
    }
}

/// The second projection is monotone for the componentwise order.
theorem pair_second_monotone_for_lte[A: PartialOrder, B: PartialOrder](p: Pair[A, B], q: Pair[A, B]) {
    pair_lte(p, q) implies p.second <= q.second
} by {
    if pair_lte(p, q) {
        pair_lte_second(p, q)
    }
}

/// Mapping both components by monotone maps preserves componentwise order.
theorem pair_map_preserves_lte[A: PartialOrder, B: PartialOrder, C: PartialOrder, D: PartialOrder](
    f: A -> C,
    g: B -> D,
    p: Pair[A, B],
    q: Pair[A, B]
) {
    is_monotone(f) and is_monotone(g) and pair_lte(p, q) implies
    pair_lte(pair_map(f, g, p), pair_map(f, g, q))
} by {
    if is_monotone(f) and is_monotone(g) and pair_lte(p, q) {
        pair_lte_first(p, q)
        p.first <= q.first
        monotone_step(f, p.first, q.first)
        f(p.first) <= f(q.first)
        pair_lte_second(p, q)
        p.second <= q.second
        monotone_step(g, p.second, q.second)
        g(p.second) <= g(q.second)
        pair_map(f, g, p).first = f(p.first)
        pair_map(f, g, q).first = f(q.first)
        pair_map(f, g, p).second = g(p.second)
        pair_map(f, g, q).second = g(q.second)
        pair_lte_of_components(pair_map(f, g, p), pair_map(f, g, q))
        pair_lte(pair_map(f, g, p), pair_map(f, g, q))
    }
}

/// Mapping both components by antitone maps reverses componentwise order.
theorem pair_map_antitone_reverses_lte[A: PartialOrder, B: PartialOrder, C: PartialOrder, D: PartialOrder](
    f: A -> C,
    g: B -> D,
    p: Pair[A, B],
    q: Pair[A, B]
) {
    is_antitone(f) and is_antitone(g) and pair_lte(p, q) implies
    pair_lte(pair_map(f, g, q), pair_map(f, g, p))
} by {
    if is_antitone(f) and is_antitone(g) and pair_lte(p, q) {
        pair_lte_first(p, q)
        p.first <= q.first
        antitone_step(f, p.first, q.first)
        f(q.first) <= f(p.first)
        pair_lte_second(p, q)
        p.second <= q.second
        antitone_step(g, p.second, q.second)
        g(q.second) <= g(p.second)
        pair_map(f, g, q).first = f(q.first)
        pair_map(f, g, p).first = f(p.first)
        pair_map(f, g, q).second = g(q.second)
        pair_map(f, g, p).second = g(p.second)
        pair_lte_of_components(pair_map(f, g, q), pair_map(f, g, p))
        pair_lte(pair_map(f, g, q), pair_map(f, g, p))
    }
}

/// Mapping both components by order embeddings reflects componentwise order.
theorem pair_map_reflects_lte_of_embeddings[A: PartialOrder, B: PartialOrder, C: PartialOrder, D: PartialOrder](
    f: A -> C,
    g: B -> D,
    p: Pair[A, B],
    q: Pair[A, B]
) {
    is_order_embedding(f) and is_order_embedding(g) and
    pair_lte(pair_map(f, g, p), pair_map(f, g, q)) implies pair_lte(p, q)
} by {
    if is_order_embedding(f) and is_order_embedding(g) and
        pair_lte(pair_map(f, g, p), pair_map(f, g, q)) {
        pair_lte_first(pair_map(f, g, p), pair_map(f, g, q))
        pair_map(f, g, p).first <= pair_map(f, g, q).first
        pair_map(f, g, p).first = f(p.first)
        pair_map(f, g, q).first = f(q.first)
        f(p.first) <= f(q.first)
        order_embedding_reflects_lte(f, p.first, q.first)
        p.first <= q.first
        pair_lte_second(pair_map(f, g, p), pair_map(f, g, q))
        pair_map(f, g, p).second <= pair_map(f, g, q).second
        pair_map(f, g, p).second = g(p.second)
        pair_map(f, g, q).second = g(q.second)
        g(p.second) <= g(q.second)
        order_embedding_reflects_lte(g, p.second, q.second)
        p.second <= q.second
        pair_lte_of_components(p, q)
        pair_lte(p, q)
    }
}

/// Mapping both components by order embeddings preserves and reflects componentwise order.
theorem pair_map_lte_iff_of_embeddings[A: PartialOrder, B: PartialOrder, C: PartialOrder, D: PartialOrder](
    f: A -> C,
    g: B -> D,
    p: Pair[A, B],
    q: Pair[A, B]
) {
    is_order_embedding(f) and is_order_embedding(g) implies
    (pair_lte(pair_map(f, g, p), pair_map(f, g, q)) = pair_lte(p, q))
} by {
    if is_order_embedding(f) and is_order_embedding(g) {
        if pair_lte(pair_map(f, g, p), pair_map(f, g, q)) {
            pair_map_reflects_lte_of_embeddings(f, g, p, q)
            pair_lte(p, q)
        }
        if pair_lte(p, q) {
            order_embedding_apply_le(f, p.first, q.first)
            f(p.first) <= f(q.first)
            order_embedding_apply_le(g, p.second, q.second)
            g(p.second) <= g(q.second)
            pair_map(f, g, p).first = f(p.first)
            pair_map(f, g, q).first = f(q.first)
            pair_map(f, g, p).second = g(p.second)
            pair_map(f, g, q).second = g(q.second)
            pair_lte_of_components(pair_map(f, g, p), pair_map(f, g, q))
            pair_lte(pair_map(f, g, p), pair_map(f, g, q))
        }
        pair_lte(pair_map(f, g, p), pair_map(f, g, q)) = pair_lte(p, q)
    }
}

/// Mapping both components by order embeddings preserves strict componentwise order.
theorem pair_map_preserves_lt_of_embeddings[A: PartialOrder, B: PartialOrder, C: PartialOrder, D: PartialOrder](
    f: A -> C,
    g: B -> D,
    p: Pair[A, B],
    q: Pair[A, B]
) {
    is_order_embedding(f) and is_order_embedding(g) and pair_lt(p, q) implies
    pair_lt(pair_map(f, g, p), pair_map(f, g, q))
} by {
    if is_order_embedding(f) and is_order_embedding(g) and pair_lt(p, q) {
        pair_lt_imp_lte(p, q)
        pair_lte(p, q)
        pair_map_lte_iff_of_embeddings(f, g, p, q)
        pair_lte(pair_map(f, g, p), pair_map(f, g, q))
        if pair_map(f, g, p) = pair_map(f, g, q) {
            pair_lte_refl(pair_map(f, g, p))
            pair_lte(pair_map(f, g, p), pair_map(f, g, p))
            pair_lte(pair_map(f, g, q), pair_map(f, g, p))
            pair_map_reflects_lte_of_embeddings(f, g, q, p)
            pair_lte(q, p)
            pair_lte_antisymm(p, q)
            p = q
            pair_lt_imp_ne(p, q)
            p != q
            false
        }
        pair_map(f, g, p) != pair_map(f, g, q)
        pair_lt_of_lte_of_ne(pair_map(f, g, p), pair_map(f, g, q))
        pair_lt(pair_map(f, g, p), pair_map(f, g, q))
    }
}

/// Mapping both components by order embeddings reflects strict componentwise order.
theorem pair_map_reflects_lt_of_embeddings[A: PartialOrder, B: PartialOrder, C: PartialOrder, D: PartialOrder](
    f: A -> C,
    g: B -> D,
    p: Pair[A, B],
    q: Pair[A, B]
) {
    is_order_embedding(f) and is_order_embedding(g) and
    pair_lt(pair_map(f, g, p), pair_map(f, g, q)) implies pair_lt(p, q)
} by {
    if is_order_embedding(f) and is_order_embedding(g) and
        pair_lt(pair_map(f, g, p), pair_map(f, g, q)) {
        pair_lt_imp_lte(pair_map(f, g, p), pair_map(f, g, q))
        pair_lte(pair_map(f, g, p), pair_map(f, g, q))
        pair_map_reflects_lte_of_embeddings(f, g, p, q)
        pair_lte(p, q)
        if p = q {
            pair_map(f, g, p) = pair_map(f, g, q)
            pair_lt_imp_ne(pair_map(f, g, p), pair_map(f, g, q))
            pair_map(f, g, p) != pair_map(f, g, q)
            false
        }
        p != q
        pair_lt_of_lte_of_ne(p, q)
        pair_lt(p, q)
    }
}

/// Mapping both components by order embeddings preserves and reflects strict componentwise order.
theorem pair_map_lt_iff_of_embeddings[A: PartialOrder, B: PartialOrder, C: PartialOrder, D: PartialOrder](
    f: A -> C,
    g: B -> D,
    p: Pair[A, B],
    q: Pair[A, B]
) {
    is_order_embedding(f) and is_order_embedding(g) implies
    (pair_lt(pair_map(f, g, p), pair_map(f, g, q)) = pair_lt(p, q))
} by {
    if is_order_embedding(f) and is_order_embedding(g) {
        if pair_lt(pair_map(f, g, p), pair_map(f, g, q)) {
            pair_map_reflects_lt_of_embeddings(f, g, p, q)
            pair_lt(p, q)
        }
        if pair_lt(p, q) {
            pair_map_preserves_lt_of_embeddings(f, g, p, q)
            pair_lt(pair_map(f, g, p), pair_map(f, g, q))
        }
        pair_lt(pair_map(f, g, p), pair_map(f, g, q)) = pair_lt(p, q)
    }
}

/// Mapping the first component by a monotone map preserves componentwise order.
theorem pair_map_first_preserves_lte[A: PartialOrder, B: PartialOrder, C: PartialOrder](
    f: A -> C,
    p: Pair[A, B],
    q: Pair[A, B]
) {
    is_monotone(f) and pair_lte(p, q) implies
    pair_lte(pair_map_first(f, p), pair_map_first(f, q))
} by {
    if is_monotone(f) and pair_lte(p, q) {
        pair_lte_first(p, q)
        monotone_step(f, p.first, q.first)
        f(p.first) <= f(q.first)
        pair_lte_second(p, q)
        p.second <= q.second
        pair_map_first(f, p).first = f(p.first)
        pair_map_first(f, q).first = f(q.first)
        pair_map_first(f, p).second = p.second
        pair_map_first(f, q).second = q.second
        pair_lte_of_components(pair_map_first(f, p), pair_map_first(f, q))
        pair_lte(pair_map_first(f, p), pair_map_first(f, q))
    }
}

/// Mapping the first component by an order embedding preserves strict componentwise order.
theorem pair_map_first_preserves_lt_of_embedding[A: PartialOrder, B: PartialOrder, C: PartialOrder](
    f: A -> C,
    p: Pair[A, B],
    q: Pair[A, B]
) {
    is_order_embedding(f) and pair_lt(p, q) implies
    pair_lt(pair_map_first(f, p), pair_map_first(f, q))
} by {
    if is_order_embedding(f) and pair_lt(p, q) {
        pair_lt_imp_lte(p, q)
        pair_lte(p, q)
        pair_map_first_preserves_lte(f, p, q)
        pair_lte(pair_map_first(f, p), pair_map_first(f, q))
        if pair_map_first(f, p) = pair_map_first(f, q) {
            pair_map_first(f, p).first = pair_map_first(f, q).first
            pair_map_first(f, p).first = f(p.first)
            pair_map_first(f, q).first = f(q.first)
            f(p.first) = f(q.first)
            lte_refl(f(p.first))
            f(p.first) <= f(q.first)
            order_embedding_reflects_lte(f, p.first, q.first)
            p.first <= q.first
            f(q.first) <= f(p.first)
            order_embedding_reflects_lte(f, q.first, p.first)
            q.first <= p.first
            pair_lte_second(p, q)
            p.second <= q.second
            pair_map_first(f, p).second = pair_map_first(f, q).second
            pair_map_first(f, p).second = p.second
            pair_map_first(f, q).second = q.second
            p.second = q.second
            pair_lte_refl(q)
            q.second <= p.second
            pair_lte_of_components(q, p)
            pair_lte(q, p)
            pair_lte_antisymm(p, q)
            p = q
            pair_lt_imp_ne(p, q)
            p != q
            false
        }
        pair_map_first(f, p) != pair_map_first(f, q)
        pair_lt_of_lte_of_ne(pair_map_first(f, p), pair_map_first(f, q))
        pair_lt(pair_map_first(f, p), pair_map_first(f, q))
    }
}

/// Mapping the second component by a monotone map preserves componentwise order.
theorem pair_map_second_preserves_lte[A: PartialOrder, B: PartialOrder, C: PartialOrder](
    f: B -> C,
    p: Pair[A, B],
    q: Pair[A, B]
) {
    is_monotone(f) and pair_lte(p, q) implies
    pair_lte(pair_map_second(f, p), pair_map_second(f, q))
} by {
    if is_monotone(f) and pair_lte(p, q) {
        pair_lte_first(p, q)
        p.first <= q.first
        pair_lte_second(p, q)
        monotone_step(f, p.second, q.second)
        f(p.second) <= f(q.second)
        pair_map_second(f, p).first = p.first
        pair_map_second(f, q).first = q.first
        pair_map_second(f, p).second = f(p.second)
        pair_map_second(f, q).second = f(q.second)
        pair_lte_of_components(pair_map_second(f, p), pair_map_second(f, q))
        pair_lte(pair_map_second(f, p), pair_map_second(f, q))
    }
}

/// Mapping the second component by an order embedding preserves strict componentwise order.
theorem pair_map_second_preserves_lt_of_embedding[A: PartialOrder, B: PartialOrder, C: PartialOrder](
    f: B -> C,
    p: Pair[A, B],
    q: Pair[A, B]
) {
    is_order_embedding(f) and pair_lt(p, q) implies
    pair_lt(pair_map_second(f, p), pair_map_second(f, q))
} by {
    if is_order_embedding(f) and pair_lt(p, q) {
        pair_lt_imp_lte(p, q)
        pair_lte(p, q)
        pair_map_second_preserves_lte(f, p, q)
        pair_lte(pair_map_second(f, p), pair_map_second(f, q))
        if pair_map_second(f, p) = pair_map_second(f, q) {
            pair_map_second(f, p).second = pair_map_second(f, q).second
            pair_map_second(f, p).second = f(p.second)
            pair_map_second(f, q).second = f(q.second)
            f(p.second) = f(q.second)
            lte_refl(f(p.second))
            f(p.second) <= f(q.second)
            order_embedding_reflects_lte(f, p.second, q.second)
            p.second <= q.second
            f(q.second) <= f(p.second)
            order_embedding_reflects_lte(f, q.second, p.second)
            q.second <= p.second
            pair_lte_first(p, q)
            p.first <= q.first
            pair_map_second(f, p).first = pair_map_second(f, q).first
            pair_map_second(f, p).first = p.first
            pair_map_second(f, q).first = q.first
            p.first = q.first
            pair_lte_refl(q)
            q.first <= p.first
            pair_lte_of_components(q, p)
            pair_lte(q, p)
            pair_lte_antisymm(p, q)
            p = q
            pair_lt_imp_ne(p, q)
            p != q
            false
        }
        pair_map_second(f, p) != pair_map_second(f, q)
        pair_lt_of_lte_of_ne(pair_map_second(f, p), pair_map_second(f, q))
        pair_lt(pair_map_second(f, p), pair_map_second(f, q))
    }
}

/// Mapping the first component by an order embedding reflects componentwise order.
theorem pair_map_first_reflects_lte_of_embedding[A: PartialOrder, B: PartialOrder, C: PartialOrder](
    f: A -> C,
    p: Pair[A, B],
    q: Pair[A, B]
) {
    is_order_embedding(f) and pair_lte(pair_map_first(f, p), pair_map_first(f, q)) implies
    pair_lte(p, q)
} by {
    if is_order_embedding(f) and pair_lte(pair_map_first(f, p), pair_map_first(f, q)) {
        pair_lte_first(pair_map_first(f, p), pair_map_first(f, q))
        pair_map_first(f, p).first = f(p.first)
        pair_map_first(f, q).first = f(q.first)
        f(p.first) <= f(q.first)
        order_embedding_reflects_lte(f, p.first, q.first)
        p.first <= q.first
        pair_lte_second(pair_map_first(f, p), pair_map_first(f, q))
        pair_map_first(f, p).second = p.second
        pair_map_first(f, q).second = q.second
        p.second <= q.second
        pair_lte_of_components(p, q)
        pair_lte(p, q)
    }
}

/// Mapping the first component by an order embedding gives an iff for componentwise order.
theorem pair_map_first_lte_iff_of_embedding[A: PartialOrder, B: PartialOrder, C: PartialOrder](
    f: A -> C,
    p: Pair[A, B],
    q: Pair[A, B]
) {
    is_order_embedding(f) implies
    (pair_lte(pair_map_first(f, p), pair_map_first(f, q)) = pair_lte(p, q))
} by {
    if is_order_embedding(f) {
        pair_map_first_preserves_lte(f, p, q)
        pair_map_first_reflects_lte_of_embedding(f, p, q)
    }
}

/// Mapping the second component by an order embedding reflects componentwise order.
theorem pair_map_second_reflects_lte_of_embedding[A: PartialOrder, B: PartialOrder, C: PartialOrder](
    f: B -> C,
    p: Pair[A, B],
    q: Pair[A, B]
) {
    is_order_embedding(f) and pair_lte(pair_map_second(f, p), pair_map_second(f, q)) implies
    pair_lte(p, q)
} by {
    if is_order_embedding(f) and pair_lte(pair_map_second(f, p), pair_map_second(f, q)) {
        pair_lte_first(pair_map_second(f, p), pair_map_second(f, q))
        pair_map_second(f, p).first = p.first
        pair_map_second(f, q).first = q.first
        p.first <= q.first
        pair_lte_second(pair_map_second(f, p), pair_map_second(f, q))
        pair_map_second(f, p).second = f(p.second)
        pair_map_second(f, q).second = f(q.second)
        f(p.second) <= f(q.second)
        order_embedding_reflects_lte(f, p.second, q.second)
        p.second <= q.second
        pair_lte_of_components(p, q)
        pair_lte(p, q)
    }
}

/// Mapping the second component by an order embedding gives an iff for componentwise order.
theorem pair_map_second_lte_iff_of_embedding[A: PartialOrder, B: PartialOrder, C: PartialOrder](
    f: B -> C,
    p: Pair[A, B],
    q: Pair[A, B]
) {
    is_order_embedding(f) implies
    (pair_lte(pair_map_second(f, p), pair_map_second(f, q)) = pair_lte(p, q))
} by {
    if is_order_embedding(f) {
        pair_map_second_preserves_lte(f, p, q)
        pair_map_second_reflects_lte_of_embedding(f, p, q)
    }
}

/// Swapping both sides converts componentwise order to componentwise order in the swapped product.
theorem pair_swap_lte_swap[A: PartialOrder, B: PartialOrder](p: Pair[A, B], q: Pair[A, B]) {
    pair_lte(p, q) implies pair_lte(p.swap, q.swap)
} by {
    if pair_lte(p, q) {
        pair_lte_first(p, q)
        p.first <= q.first
        pair_lte_second(p, q)
        p.second <= q.second
        p.swap.first = p.second
        q.swap.first = q.second
        p.swap.second = p.first
        q.swap.second = q.first
        pair_lte_of_components(p.swap, q.swap)
        pair_lte(p.swap, q.swap)
    }
}

/// Swapping both sides converts strict componentwise order to strict componentwise order in the swapped product.
theorem pair_swap_lt_swap[A: PartialOrder, B: PartialOrder](p: Pair[A, B], q: Pair[A, B]) {
    pair_lt(p, q) implies pair_lt(p.swap, q.swap)
} by {
    if pair_lt(p, q) {
        pair_lt_imp_lte(p, q)
        pair_lte(p, q)
        pair_swap_lte_swap(p, q)
        pair_lte(p.swap, q.swap)
        if p.swap = q.swap {
            p.swap.swap = q.swap.swap
            swap_swap(p)
            swap_swap(q)
            p = q
            pair_lt_imp_ne(p, q)
            p != q
            false
        }
        p.swap != q.swap
        pair_lt_of_lte_of_ne(p.swap, q.swap)
        pair_lt(p.swap, q.swap)
    }
}

/// Swapping both sides reflects componentwise order.
theorem pair_swap_reflects_lte[A: PartialOrder, B: PartialOrder](p: Pair[A, B], q: Pair[A, B]) {
    pair_lte(p.swap, q.swap) implies pair_lte(p, q)
} by {
    if pair_lte(p.swap, q.swap) {
        pair_swap_lte_swap(p.swap, q.swap)
        pair_lte(p.swap.swap, q.swap.swap)
        swap_swap(p)
        swap_swap(q)
        pair_lte(p, q)
    }
}

/// Swapping both sides reflects strict componentwise order.
theorem pair_swap_reflects_lt[A: PartialOrder, B: PartialOrder](p: Pair[A, B], q: Pair[A, B]) {
    pair_lt(p.swap, q.swap) implies pair_lt(p, q)
} by {
    if pair_lt(p.swap, q.swap) {
        pair_swap_lt_swap(p.swap, q.swap)
        pair_lt(p.swap.swap, q.swap.swap)
        swap_swap(p)
        swap_swap(q)
        pair_lt(p, q)
    }
}

/// Swapping a product is an order embedding.
theorem pair_swap_is_order_embedding[A: PartialOrder, B: PartialOrder] {
    forall(p: Pair[A, B], q: Pair[A, B]) {
        pair_lte(p.swap, q.swap) = pair_lte(p, q)
    }
} by {
    forall(p: Pair[A, B], q: Pair[A, B]) {
        if pair_lte(p.swap, q.swap) {
            pair_swap_reflects_lte(p, q)
            pair_lte(p, q)
        }
        if pair_lte(p, q) {
            pair_swap_lte_swap(p, q)
            pair_lte(p.swap, q.swap)
        }
        pair_lte(p.swap, q.swap) = pair_lte(p, q)
    }
}

/// Swapped pairs are ordered exactly when the original pairs are ordered.
theorem pair_swap_lte_iff[A: PartialOrder, B: PartialOrder](p: Pair[A, B], q: Pair[A, B]) {
    pair_lte(p.swap, q.swap) = pair_lte(p, q)
} by {
    pair_swap_is_order_embedding[A, B]
}

/// Swapped pairs are strictly ordered exactly when the original pairs are strictly ordered.
theorem pair_swap_lt_iff[A: PartialOrder, B: PartialOrder](p: Pair[A, B], q: Pair[A, B]) {
    pair_lt(p.swap, q.swap) = pair_lt(p, q)
} by {
    if pair_lt(p.swap, q.swap) {
        pair_swap_reflects_lt(p, q)
        pair_lt(p, q)
    }
    if pair_lt(p, q) {
        pair_swap_lt_swap(p, q)
        pair_lt(p.swap, q.swap)
    }
    pair_lt(p.swap, q.swap) = pair_lt(p, q)
}

/// The diagonal pair preserves non-strict order.
theorem pair_diag_preserves_lte[A: PartialOrder](a: A, b: A) {
    a <= b implies pair_lte(Pair.new(a, a), Pair.new(b, b))
} by {
    if a <= b {
        pair_new_lte_iff(a, a, b, b)
        pair_lte(Pair.new(a, a), Pair.new(b, b))
    }
}

/// The diagonal pair reflects non-strict order.
theorem pair_diag_reflects_lte[A: PartialOrder](a: A, b: A) {
    pair_lte(Pair.new(a, a), Pair.new(b, b)) implies a <= b
} by {
    if pair_lte(Pair.new(a, a), Pair.new(b, b)) {
        pair_new_lte_iff(a, a, b, b)
        a <= b
    }
}

/// Diagonal pairs are ordered exactly when their underlying elements are.
theorem pair_diag_lte_iff[A: PartialOrder](a: A, b: A) {
    pair_lte(Pair.new(a, a), Pair.new(b, b)) = (a <= b)
} by {
    if pair_lte(Pair.new(a, a), Pair.new(b, b)) {
        pair_diag_reflects_lte(a, b)
        a <= b
    }
    if a <= b {
        pair_diag_preserves_lte(a, b)
        pair_lte(Pair.new(a, a), Pair.new(b, b))
    }
    pair_lte(Pair.new(a, a), Pair.new(b, b)) = (a <= b)
}

/// The diagonal pair preserves strict order.
theorem pair_diag_preserves_lt[A: PartialOrder](a: A, b: A) {
    a < b implies pair_lt(Pair.new(a, a), Pair.new(b, b))
} by {
    if a < b {
        lt_imp_lte(a, b)
        a <= b
        pair_diag_preserves_lte(a, b)
        pair_lte(Pair.new(a, a), Pair.new(b, b))
        lt_imp_ne(a, b)
        a != b
        Pair.new(a, a) != Pair.new(b, b)
        pair_lt_of_lte_of_ne(Pair.new(a, a), Pair.new(b, b))
        pair_lt(Pair.new(a, a), Pair.new(b, b))
    }
}

/// The diagonal pair reflects strict order.
theorem pair_diag_reflects_lt[A: PartialOrder](a: A, b: A) {
    pair_lt(Pair.new(a, a), Pair.new(b, b)) implies a < b
} by {
    if pair_lt(Pair.new(a, a), Pair.new(b, b)) {
        pair_lt_imp_lte(Pair.new(a, a), Pair.new(b, b))
        pair_diag_reflects_lte(a, b)
        a <= b
        pair_lt_imp_ne(Pair.new(a, a), Pair.new(b, b))
        Pair.new(a, a) != Pair.new(b, b)
        if a = b {
            Pair.new(a, a) = Pair.new(b, b)
            false
        }
        a != b
    }
}

/// Diagonal pairs are strictly ordered exactly when their underlying elements are.
theorem pair_diag_lt_iff[A: PartialOrder](a: A, b: A) {
    pair_lt(Pair.new(a, a), Pair.new(b, b)) = (a < b)
} by {
    if pair_lt(Pair.new(a, a), Pair.new(b, b)) {
        pair_diag_reflects_lt(a, b)
        a < b
    }
    if a < b {
        pair_diag_preserves_lt(a, b)
        pair_lt(Pair.new(a, a), Pair.new(b, b))
    }
    pair_lt(Pair.new(a, a), Pair.new(b, b)) = (a < b)
}

/// Strict componentwise pair order forces some component to be strictly below.
theorem pair_lt_imp_some_strict_component[A: PartialOrder, B: PartialOrder](p: Pair[A, B], q: Pair[A, B]) {
    pair_lt(p, q) implies (p.first < q.first or p.second < q.second)
} by {
    if pair_lt(p, q) {
        pair_lt_imp_lte(p, q)
        pair_lte_first(p, q)
        p.first <= q.first
        pair_lte_second(p, q)
        p.second <= q.second
        pair_lt_imp_ne(p, q)
        p != q
        if p.first = q.first and p.second = q.second {
            pair_ext(p, q)
            p = q
            false
        }
        p.first != q.first or p.second != q.second
        if p.first != q.first {
            lt_of_lte_of_ne(p.first, q.first)
            p.first < q.first
            p.first < q.first or p.second < q.second
        }
        if p.second != q.second {
            lt_of_lte_of_ne(p.second, q.second)
            p.second < q.second
            p.first < q.first or p.second < q.second
        }
        p.first < q.first or p.second < q.second
    }
}

/// Strict componentwise pair order decomposes into a single-strict-component disjunction (forward direction).
theorem pair_lt_imp_one_strict_component[A: PartialOrder, B: PartialOrder](p: Pair[A, B], q: Pair[A, B]) {
    pair_lt(p, q) implies ((p.first < q.first and p.second <= q.second) or (p.first <= q.first and p.second < q.second))
} by {
    if pair_lt(p, q) {
        pair_lt_imp_lte(p, q)
        pair_lte_first(p, q)
        p.first <= q.first
        pair_lte_second(p, q)
        p.second <= q.second
        pair_lt_imp_some_strict_component(p, q)
        if p.first < q.first {
            p.first < q.first and p.second <= q.second
            (p.first < q.first and p.second <= q.second) or (p.first <= q.first and p.second < q.second)
        }
        if p.second < q.second {
            p.first <= q.first and p.second < q.second
            (p.first < q.first and p.second <= q.second) or (p.first <= q.first and p.second < q.second)
        }
        (p.first < q.first and p.second <= q.second) or (p.first <= q.first and p.second < q.second)
    }
}

/// A single strict component combined with the other non-strict gives strict pair order (converse direction).
theorem pair_lt_of_one_strict_component[A: PartialOrder, B: PartialOrder](p: Pair[A, B], q: Pair[A, B]) {
    ((p.first < q.first and p.second <= q.second) or (p.first <= q.first and p.second < q.second))
        implies pair_lt(p, q)
} by {
    let caseA: Bool = p.first < q.first and p.second <= q.second
    let caseB: Bool = p.first <= q.first and p.second < q.second
    if caseA {
        pair_lt_of_first_lt_of_second_lte(p, q)
        pair_lt(p, q)
    }
    if caseB {
        pair_lt_of_first_lte_of_second_lt(p, q)
        pair_lt(p, q)
    }
}

/// Mapping both components, one monotone and one antitone, reverses on the second component.
theorem pair_map_mixed_first_monotone_second_antitone[A: PartialOrder, B: PartialOrder, C: PartialOrder, D: PartialOrder](
    f: A -> C,
    g: B -> D,
    p: Pair[A, B],
    q: Pair[A, B]
) {
    is_monotone(f) and is_antitone(g) and pair_lte(p, q) implies
    f(p.first) <= f(q.first) and g(q.second) <= g(p.second)
} by {
    if is_monotone(f) and is_antitone(g) and pair_lte(p, q) {
        pair_lte_first(p, q)
        p.first <= q.first
        monotone_step(f, p.first, q.first)
        f(p.first) <= f(q.first)
        pair_lte_second(p, q)
        p.second <= q.second
        antitone_step(g, p.second, q.second)
        g(q.second) <= g(p.second)
        f(p.first) <= f(q.first) and g(q.second) <= g(p.second)
    }
}

/// Mapping both components, one antitone and one monotone, reverses on the first component.
theorem pair_map_mixed_first_antitone_second_monotone[A: PartialOrder, B: PartialOrder, C: PartialOrder, D: PartialOrder](
    f: A -> C,
    g: B -> D,
    p: Pair[A, B],
    q: Pair[A, B]
) {
    is_antitone(f) and is_monotone(g) and pair_lte(p, q) implies
    f(q.first) <= f(p.first) and g(p.second) <= g(q.second)
} by {
    if is_antitone(f) and is_monotone(g) and pair_lte(p, q) {
        pair_lte_first(p, q)
        p.first <= q.first
        antitone_step(f, p.first, q.first)
        f(q.first) <= f(p.first)
        pair_lte_second(p, q)
        p.second <= q.second
        monotone_step(g, p.second, q.second)
        g(p.second) <= g(q.second)
        f(q.first) <= f(p.first) and g(p.second) <= g(q.second)
    }
}

/// Mapping both components by antitone maps reverses strict componentwise order.
theorem pair_map_antitone_reverses_lt[A: PartialOrder, B: PartialOrder, C: PartialOrder, D: PartialOrder](
    f: A -> C,
    g: B -> D,
    p: Pair[A, B],
    q: Pair[A, B]
) {
    is_antitone(f) and is_antitone(g) and is_order_embedding(f) and is_order_embedding(g) and pair_lt(p, q) implies
    pair_lt(pair_map(f, g, q), pair_map(f, g, p))
} by {
    if is_antitone(f) and is_antitone(g) and is_order_embedding(f) and is_order_embedding(g) and pair_lt(p, q) {
        pair_lt_imp_lte(p, q)
        pair_lte(p, q)
        pair_map_antitone_reverses_lte(f, g, p, q)
        pair_lte(pair_map(f, g, q), pair_map(f, g, p))
        if pair_map(f, g, q) = pair_map(f, g, p) {
            pair_map(f, g, q).first = pair_map(f, g, p).first
            pair_map(f, g, q).first = f(q.first)
            pair_map(f, g, p).first = f(p.first)
            f(q.first) = f(p.first)
            lte_refl(f(q.first))
            f(q.first) <= f(p.first)
            f(p.first) <= f(q.first)
            order_embedding_reflects_lte(f, p.first, q.first)
            p.first <= q.first
            order_embedding_reflects_lte(f, q.first, p.first)
            q.first <= p.first
            antisymmetric_eq(A.lte, p.first, q.first)
            p.first = q.first
            pair_map(f, g, q).second = pair_map(f, g, p).second
            pair_map(f, g, q).second = g(q.second)
            pair_map(f, g, p).second = g(p.second)
            g(q.second) = g(p.second)
            lte_refl(g(q.second))
            g(q.second) <= g(p.second)
            g(p.second) <= g(q.second)
            order_embedding_reflects_lte(g, p.second, q.second)
            p.second <= q.second
            order_embedding_reflects_lte(g, q.second, p.second)
            q.second <= p.second
            antisymmetric_eq(B.lte, p.second, q.second)
            p.second = q.second
            pair_ext(p, q)
            p = q
            pair_lt_imp_ne(p, q)
            p != q
            false
        }
        pair_map(f, g, q) != pair_map(f, g, p)
        pair_lt_of_lte_of_ne(pair_map(f, g, q), pair_map(f, g, p))
        pair_lt(pair_map(f, g, q), pair_map(f, g, p))
    }
}

/// Strict componentwise pair order is non-strict pair order with at least one strict component.
theorem pair_lt_iff_lte_and_some_strict_component[A: PartialOrder, B: PartialOrder](p: Pair[A, B], q: Pair[A, B]) {
    pair_lt(p, q) = (pair_lte(p, q) and (p.first < q.first or p.second < q.second))
} by {
    if pair_lt(p, q) {
        pair_lt_imp_lte(p, q)
        pair_lte(p, q)
        pair_lt_imp_some_strict_component(p, q)
        p.first < q.first or p.second < q.second
        pair_lte(p, q) and (p.first < q.first or p.second < q.second)
    }
    if pair_lte(p, q) and (p.first < q.first or p.second < q.second) {
        pair_lte_first(p, q)
        p.first <= q.first
        pair_lte_second(p, q)
        p.second <= q.second
        if p.first < q.first {
            pair_lt_of_first_lt_of_second_lte(p, q)
            pair_lt(p, q)
        }
        if p.second < q.second {
            pair_lt_of_first_lte_of_second_lt(p, q)
            pair_lt(p, q)
        }
        pair_lt(p, q)
    }
}

from lattice import Meet, Join, MeetSemilattice, JoinSemilattice, Lattice, DistribLattice, meet_lte_left, meet_lte_right,
    lte_meet_of_bounds, lte_join_left, lte_join_right, join_lte_of_bounds,
    meet_lte_meet, join_lte_join, meet_absorb_join, join_absorb_meet,
    meet_absorb_join_left, join_absorb_meet_left,
    meet_join_distrib_left, join_meet_distrib_left,
    meet_join_distrib_right, join_meet_distrib_right

/// Componentwise meet on pairs of meet semilattices.
define pair_meet[A: MeetSemilattice, B: MeetSemilattice](p: Pair[A, B], q: Pair[A, B]) -> Pair[A, B] {
    Pair.new(p.first.meet(q.first), p.second.meet(q.second))
}

/// Componentwise join on pairs of join semilattices.
define pair_join[A: JoinSemilattice, B: JoinSemilattice](p: Pair[A, B], q: Pair[A, B]) -> Pair[A, B] {
    Pair.new(p.first.join(q.first), p.second.join(q.second))
}

/// The first component of a pair meet is the meet of first components.
theorem pair_meet_first[A: MeetSemilattice, B: MeetSemilattice](p: Pair[A, B], q: Pair[A, B]) {
    pair_meet(p, q).first = p.first.meet(q.first)
}

/// The second component of a pair meet is the meet of second components.
theorem pair_meet_second[A: MeetSemilattice, B: MeetSemilattice](p: Pair[A, B], q: Pair[A, B]) {
    pair_meet(p, q).second = p.second.meet(q.second)
}

/// The first component of a pair join is the join of first components.
theorem pair_join_first[A: JoinSemilattice, B: JoinSemilattice](p: Pair[A, B], q: Pair[A, B]) {
    pair_join(p, q).first = p.first.join(q.first)
}

/// The second component of a pair join is the join of second components.
theorem pair_join_second[A: JoinSemilattice, B: JoinSemilattice](p: Pair[A, B], q: Pair[A, B]) {
    pair_join(p, q).second = p.second.join(q.second)
}

/// A pair meet is below its left argument.
theorem pair_meet_lte_left[A: MeetSemilattice, B: MeetSemilattice](p: Pair[A, B], q: Pair[A, B]) {
    pair_lte(pair_meet(p, q), p)
} by {
    pair_meet_first(p, q)
    pair_meet_second(p, q)
    p.first.meet(q.first) <= p.first
    p.second.meet(q.second) <= p.second
    pair_meet(p, q).first <= p.first
    pair_meet(p, q).second <= p.second
    pair_lte_of_components(pair_meet(p, q), p)
}

/// A pair meet is below its right argument.
theorem pair_meet_lte_right[A: MeetSemilattice, B: MeetSemilattice](p: Pair[A, B], q: Pair[A, B]) {
    pair_lte(pair_meet(p, q), q)
} by {
    pair_meet_first(p, q)
    pair_meet_second(p, q)
    p.first.meet(q.first) <= q.first
    p.second.meet(q.second) <= q.second
    pair_meet(p, q).first <= q.first
    pair_meet(p, q).second <= q.second
    pair_lte_of_components(pair_meet(p, q), q)
}

/// Any common pair lower bound lies below the pair meet.
theorem pair_lte_meet_of_bounds[A: MeetSemilattice, B: MeetSemilattice](
    c: Pair[A, B], p: Pair[A, B], q: Pair[A, B]
) {
    pair_lte(c, p) and pair_lte(c, q) implies pair_lte(c, pair_meet(p, q))
} by {
    if pair_lte(c, p) and pair_lte(c, q) {
        pair_lte_first(c, p)
        pair_lte_first(c, q)
        c.first <= p.first
        c.first <= q.first
        lte_meet_of_bounds(c.first, p.first, q.first)
        c.first <= p.first.meet(q.first)
        pair_lte_second(c, p)
        pair_lte_second(c, q)
        c.second <= p.second
        c.second <= q.second
        lte_meet_of_bounds(c.second, p.second, q.second)
        c.second <= p.second.meet(q.second)
        pair_meet_first(p, q)
        pair_meet_second(p, q)
        c.first <= pair_meet(p, q).first
        c.second <= pair_meet(p, q).second
        c.first <= pair_meet(p, q).first and c.second <= pair_meet(p, q).second
        pair_lte_of_components(c, pair_meet(p, q))
        pair_lte(c, pair_meet(p, q))
    }
}

/// The pair meet is the greatest lower bound under componentwise order.
theorem pair_lte_meet_iff[A: MeetSemilattice, B: MeetSemilattice](
    c: Pair[A, B], p: Pair[A, B], q: Pair[A, B]
) {
    pair_lte(c, pair_meet(p, q)) = (pair_lte(c, p) and pair_lte(c, q))
} by {
    if pair_lte(c, pair_meet(p, q)) {
        pair_meet_lte_left(p, q)
        pair_lte_trans(c, pair_meet(p, q), p)
        pair_lte(c, p)
        pair_meet_lte_right(p, q)
        pair_lte_trans(c, pair_meet(p, q), q)
        pair_lte(c, q)
        pair_lte(c, p) and pair_lte(c, q)
    }
    if pair_lte(c, p) and pair_lte(c, q) {
        pair_lte_meet_of_bounds(c, p, q)
        pair_lte(c, pair_meet(p, q))
    }
}

/// A pair join is above its left argument.
theorem pair_lte_join_left[A: JoinSemilattice, B: JoinSemilattice](p: Pair[A, B], q: Pair[A, B]) {
    pair_lte(p, pair_join(p, q))
} by {
    pair_join_first(p, q)
    pair_join_second(p, q)
    p.first <= p.first.join(q.first)
    p.second <= p.second.join(q.second)
    p.first <= pair_join(p, q).first
    p.second <= pair_join(p, q).second
    pair_lte_of_components(p, pair_join(p, q))
}

/// A pair join is above its right argument.
theorem pair_lte_join_right[A: JoinSemilattice, B: JoinSemilattice](p: Pair[A, B], q: Pair[A, B]) {
    pair_lte(q, pair_join(p, q))
} by {
    pair_join_first(p, q)
    pair_join_second(p, q)
    q.first <= p.first.join(q.first)
    q.second <= p.second.join(q.second)
    q.first <= pair_join(p, q).first
    q.second <= pair_join(p, q).second
    pair_lte_of_components(q, pair_join(p, q))
}

/// The pair join is below any common pair upper bound.
theorem pair_join_lte_of_bounds[A: JoinSemilattice, B: JoinSemilattice](
    p: Pair[A, B], q: Pair[A, B], c: Pair[A, B]
) {
    pair_lte(p, c) and pair_lte(q, c) implies pair_lte(pair_join(p, q), c)
} by {
    if pair_lte(p, c) and pair_lte(q, c) {
        pair_lte_first(p, c)
        pair_lte_first(q, c)
        p.first <= c.first
        q.first <= c.first
        join_lte_of_bounds(p.first, q.first, c.first)
        p.first.join(q.first) <= c.first
        pair_lte_second(p, c)
        pair_lte_second(q, c)
        p.second <= c.second
        q.second <= c.second
        join_lte_of_bounds(p.second, q.second, c.second)
        p.second.join(q.second) <= c.second
        pair_join_first(p, q)
        pair_join_second(p, q)
        pair_join(p, q).first <= c.first
        pair_join(p, q).second <= c.second
        pair_join(p, q).first <= c.first and pair_join(p, q).second <= c.second
        pair_lte_of_components(pair_join(p, q), c)
        pair_lte(pair_join(p, q), c)
    }
}

/// The pair join is the least upper bound under componentwise order.
theorem pair_join_lte_iff[A: JoinSemilattice, B: JoinSemilattice](
    p: Pair[A, B], q: Pair[A, B], c: Pair[A, B]
) {
    pair_lte(pair_join(p, q), c) = (pair_lte(p, c) and pair_lte(q, c))
} by {
    if pair_lte(pair_join(p, q), c) {
        pair_lte_join_left(p, q)
        pair_lte_trans(p, pair_join(p, q), c)
        pair_lte(p, c)
        pair_lte_join_right(p, q)
        pair_lte_trans(q, pair_join(p, q), c)
        pair_lte(q, c)
        pair_lte(p, c) and pair_lte(q, c)
    }
    if pair_lte(p, c) and pair_lte(q, c) {
        pair_join_lte_of_bounds(p, q, c)
        pair_lte(pair_join(p, q), c)
    }
}

/// Pair meet is commutative.
theorem pair_meet_comm[A: MeetSemilattice, B: MeetSemilattice](p: Pair[A, B], q: Pair[A, B]) {
    pair_meet(p, q) = pair_meet(q, p)
} by {
    pair_meet_first(p, q)
    pair_meet_second(p, q)
    pair_meet_first(q, p)
    pair_meet_second(q, p)
    p.first.meet(q.first) = q.first.meet(p.first)
    p.second.meet(q.second) = q.second.meet(p.second)
    pair_meet(p, q).first = pair_meet(q, p).first
    pair_meet(p, q).second = pair_meet(q, p).second
    pair_ext(pair_meet(p, q), pair_meet(q, p))
}

/// Pair join is commutative.
theorem pair_join_comm[A: JoinSemilattice, B: JoinSemilattice](p: Pair[A, B], q: Pair[A, B]) {
    pair_join(p, q) = pair_join(q, p)
} by {
    pair_join_first(p, q)
    pair_join_second(p, q)
    pair_join_first(q, p)
    pair_join_second(q, p)
    p.first.join(q.first) = q.first.join(p.first)
    p.second.join(q.second) = q.second.join(p.second)
    pair_join(p, q).first = pair_join(q, p).first
    pair_join(p, q).second = pair_join(q, p).second
    pair_ext(pair_join(p, q), pair_join(q, p))
}

/// Pair meet is idempotent.
theorem pair_meet_idem[A: MeetSemilattice, B: MeetSemilattice](p: Pair[A, B]) {
    pair_meet(p, p) = p
} by {
    pair_meet_first(p, p)
    pair_meet_second(p, p)
    p.first.meet(p.first) = p.first
    p.second.meet(p.second) = p.second
    pair_meet(p, p).first = p.first
    pair_meet(p, p).second = p.second
    pair_ext(pair_meet(p, p), p)
}

/// Pair join is idempotent.
theorem pair_join_idem[A: JoinSemilattice, B: JoinSemilattice](p: Pair[A, B]) {
    pair_join(p, p) = p
} by {
    pair_join_first(p, p)
    pair_join_second(p, p)
    p.first.join(p.first) = p.first
    p.second.join(p.second) = p.second
    pair_join(p, p).first = p.first
    pair_join(p, p).second = p.second
    pair_ext(pair_join(p, p), p)
}

/// Pair meet is associative.
theorem pair_meet_assoc[A: MeetSemilattice, B: MeetSemilattice](p: Pair[A, B], q: Pair[A, B], r: Pair[A, B]) {
    pair_meet(pair_meet(p, q), r) = pair_meet(p, pair_meet(q, r))
} by {
    pair_meet_first(pair_meet(p, q), r)
    pair_meet_second(pair_meet(p, q), r)
    pair_meet_first(p, pair_meet(q, r))
    pair_meet_second(p, pair_meet(q, r))
    pair_meet_first(p, q)
    pair_meet_second(p, q)
    pair_meet_first(q, r)
    pair_meet_second(q, r)
    p.first.meet(q.first).meet(r.first) = p.first.meet(q.first.meet(r.first))
    p.second.meet(q.second).meet(r.second) = p.second.meet(q.second.meet(r.second))
    pair_meet(pair_meet(p, q), r).first = pair_meet(p, pair_meet(q, r)).first
    pair_meet(pair_meet(p, q), r).second = pair_meet(p, pair_meet(q, r)).second
    pair_ext(pair_meet(pair_meet(p, q), r), pair_meet(p, pair_meet(q, r)))
}

/// Pair join is associative.
theorem pair_join_assoc[A: JoinSemilattice, B: JoinSemilattice](p: Pair[A, B], q: Pair[A, B], r: Pair[A, B]) {
    pair_join(pair_join(p, q), r) = pair_join(p, pair_join(q, r))
} by {
    pair_join_first(pair_join(p, q), r)
    pair_join_second(pair_join(p, q), r)
    pair_join_first(p, pair_join(q, r))
    pair_join_second(p, pair_join(q, r))
    pair_join_first(p, q)
    pair_join_second(p, q)
    pair_join_first(q, r)
    pair_join_second(q, r)
    p.first.join(q.first).join(r.first) = p.first.join(q.first.join(r.first))
    p.second.join(q.second).join(r.second) = p.second.join(q.second.join(r.second))
    pair_join(pair_join(p, q), r).first = pair_join(p, pair_join(q, r)).first
    pair_join(pair_join(p, q), r).second = pair_join(p, pair_join(q, r)).second
    pair_ext(pair_join(pair_join(p, q), r), pair_join(p, pair_join(q, r)))
}

/// A pair meet equals its left argument iff the argument is below the right.
theorem pair_meet_eq_left_iff_lte[A: MeetSemilattice, B: MeetSemilattice](p: Pair[A, B], q: Pair[A, B]) {
    pair_meet(p, q) = p = pair_lte(p, q)
} by {
    if pair_meet(p, q) = p {
        pair_meet_lte_right(p, q)
        pair_lte(pair_meet(p, q), q)
        pair_lte(p, q)
    }
    if pair_lte(p, q) {
        pair_lte_first(p, q)
        pair_lte_second(p, q)
        p.first <= q.first
        p.second <= q.second
        p.first.meet(q.first) = p.first
        p.second.meet(q.second) = p.second
        pair_meet_first(p, q)
        pair_meet_second(p, q)
        pair_meet(p, q).first = p.first
        pair_meet(p, q).second = p.second
        pair_ext(pair_meet(p, q), p)
        pair_meet(p, q) = p
    }
}

/// A pair join equals its right argument iff the left is below the right.
theorem pair_join_eq_right_iff_lte[A: JoinSemilattice, B: JoinSemilattice](p: Pair[A, B], q: Pair[A, B]) {
    pair_join(p, q) = q = pair_lte(p, q)
} by {
    if pair_join(p, q) = q {
        pair_lte_join_left(p, q)
        pair_lte(p, pair_join(p, q))
        pair_lte(p, q)
    }
    if pair_lte(p, q) {
        pair_lte_first(p, q)
        pair_lte_second(p, q)
        p.first <= q.first
        p.second <= q.second
        p.first.join(q.first) = q.first
        p.second.join(q.second) = q.second
        pair_join_first(p, q)
        pair_join_second(p, q)
        pair_join(p, q).first = q.first
        pair_join(p, q).second = q.second
        pair_ext(pair_join(p, q), q)
        pair_join(p, q) = q
    }
}

/// A pair meet equals its right argument iff the right is below the left.
theorem pair_meet_eq_right_iff_lte[A: MeetSemilattice, B: MeetSemilattice](p: Pair[A, B], q: Pair[A, B]) {
    pair_meet(p, q) = q = pair_lte(q, p)
} by {
    if pair_meet(p, q) = q {
        pair_meet_lte_left(p, q)
        pair_lte(pair_meet(p, q), p)
        pair_lte(q, p)
    }
    if pair_lte(q, p) {
        pair_lte_first(q, p)
        pair_lte_second(q, p)
        q.first <= p.first
        q.second <= p.second
        p.first.meet(q.first) = q.first
        p.second.meet(q.second) = q.second
        pair_meet_first(p, q)
        pair_meet_second(p, q)
        pair_meet(p, q).first = q.first
        pair_meet(p, q).second = q.second
        pair_ext(pair_meet(p, q), q)
        pair_meet(p, q) = q
    }
}

/// A pair join equals its left argument iff the right is below the left.
theorem pair_join_eq_left_iff_lte[A: JoinSemilattice, B: JoinSemilattice](p: Pair[A, B], q: Pair[A, B]) {
    pair_join(p, q) = p = pair_lte(q, p)
} by {
    if pair_join(p, q) = p {
        pair_lte_join_right(p, q)
        pair_lte(q, pair_join(p, q))
        pair_lte(q, p)
    }
    if pair_lte(q, p) {
        pair_lte_first(q, p)
        pair_lte_second(q, p)
        q.first <= p.first
        q.second <= p.second
        p.first.join(q.first) = p.first
        p.second.join(q.second) = p.second
        pair_join_first(p, q)
        pair_join_second(p, q)
        pair_join(p, q).first = p.first
        pair_join(p, q).second = p.second
        pair_ext(pair_join(p, q), p)
        pair_join(p, q) = p
    }
}

/// Left monotonicity of pair meet.
theorem pair_meet_lte_meet_left[A: MeetSemilattice, B: MeetSemilattice](
    p: Pair[A, B], q: Pair[A, B], r: Pair[A, B]) {
    pair_lte(p, q) implies pair_lte(pair_meet(p, r), pair_meet(q, r))
} by {
    if pair_lte(p, q) {
        pair_lte_first(p, q)
        pair_lte_second(p, q)
        p.first <= q.first
        p.second <= q.second
        lte_refl(r.first)
        lte_refl(r.second)
        meet_lte_meet(p.first, q.first, r.first, r.first)
        meet_lte_meet(p.second, q.second, r.second, r.second)
        p.first.meet(r.first) <= q.first.meet(r.first)
        p.second.meet(r.second) <= q.second.meet(r.second)
        pair_meet_first(p, r)
        pair_meet_second(p, r)
        pair_meet_first(q, r)
        pair_meet_second(q, r)
        pair_meet(p, r).first <= pair_meet(q, r).first
        pair_meet(p, r).second <= pair_meet(q, r).second
        pair_meet(p, r).first <= pair_meet(q, r).first and pair_meet(p, r).second <= pair_meet(q, r).second
        pair_lte_of_components(pair_meet(p, r), pair_meet(q, r))
    }
}

/// Right monotonicity of pair meet.
theorem pair_meet_lte_meet_right[A: MeetSemilattice, B: MeetSemilattice](
    p: Pair[A, B], q: Pair[A, B], r: Pair[A, B]) {
    pair_lte(p, q) implies pair_lte(pair_meet(r, p), pair_meet(r, q))
} by {
    if pair_lte(p, q) {
        pair_lte_first(p, q)
        pair_lte_second(p, q)
        p.first <= q.first
        p.second <= q.second
        lte_refl(r.first)
        lte_refl(r.second)
        meet_lte_meet(r.first, r.first, p.first, q.first)
        meet_lte_meet(r.second, r.second, p.second, q.second)
        r.first.meet(p.first) <= r.first.meet(q.first)
        r.second.meet(p.second) <= r.second.meet(q.second)
        pair_meet_first(r, p)
        pair_meet_second(r, p)
        pair_meet_first(r, q)
        pair_meet_second(r, q)
        pair_meet(r, p).first <= pair_meet(r, q).first
        pair_meet(r, p).second <= pair_meet(r, q).second
        pair_meet(r, p).first <= pair_meet(r, q).first and pair_meet(r, p).second <= pair_meet(r, q).second
        pair_lte_of_components(pair_meet(r, p), pair_meet(r, q))
    }
}

/// Left monotonicity of pair join.
theorem pair_join_lte_join_left[A: JoinSemilattice, B: JoinSemilattice](
    p: Pair[A, B], q: Pair[A, B], r: Pair[A, B]) {
    pair_lte(p, q) implies pair_lte(pair_join(p, r), pair_join(q, r))
} by {
    if pair_lte(p, q) {
        pair_lte_first(p, q)
        pair_lte_second(p, q)
        p.first <= q.first
        p.second <= q.second
        lte_refl(r.first)
        lte_refl(r.second)
        join_lte_join(p.first, q.first, r.first, r.first)
        join_lte_join(p.second, q.second, r.second, r.second)
        p.first.join(r.first) <= q.first.join(r.first)
        p.second.join(r.second) <= q.second.join(r.second)
        pair_join_first(p, r)
        pair_join_second(p, r)
        pair_join_first(q, r)
        pair_join_second(q, r)
        pair_join(p, r).first <= pair_join(q, r).first
        pair_join(p, r).second <= pair_join(q, r).second
        pair_join(p, r).first <= pair_join(q, r).first and pair_join(p, r).second <= pair_join(q, r).second
        pair_lte_of_components(pair_join(p, r), pair_join(q, r))
    }
}

/// Right monotonicity of pair join.
theorem pair_join_lte_join_right[A: JoinSemilattice, B: JoinSemilattice](
    p: Pair[A, B], q: Pair[A, B], r: Pair[A, B]) {
    pair_lte(p, q) implies pair_lte(pair_join(r, p), pair_join(r, q))
} by {
    if pair_lte(p, q) {
        pair_lte_first(p, q)
        pair_lte_second(p, q)
        p.first <= q.first
        p.second <= q.second
        lte_refl(r.first)
        lte_refl(r.second)
        join_lte_join(r.first, r.first, p.first, q.first)
        join_lte_join(r.second, r.second, p.second, q.second)
        r.first.join(p.first) <= r.first.join(q.first)
        r.second.join(p.second) <= r.second.join(q.second)
        pair_join_first(r, p)
        pair_join_second(r, p)
        pair_join_first(r, q)
        pair_join_second(r, q)
        pair_join(r, p).first <= pair_join(r, q).first
        pair_join(r, p).second <= pair_join(r, q).second
        pair_join(r, p).first <= pair_join(r, q).first and pair_join(r, p).second <= pair_join(r, q).second
        pair_lte_of_components(pair_join(r, p), pair_join(r, q))
    }
}

/// Pair meet is jointly monotone in both arguments.
theorem pair_meet_lte_meet[A: MeetSemilattice, B: MeetSemilattice](
    a: Pair[A, B], b: Pair[A, B], c: Pair[A, B], d: Pair[A, B]) {
    pair_lte(a, b) and pair_lte(c, d) implies pair_lte(pair_meet(a, c), pair_meet(b, d))
} by {
    if pair_lte(a, b) and pair_lte(c, d) {
        pair_meet_lte_meet_left(a, b, c)
        pair_lte(pair_meet(a, c), pair_meet(b, c))
        pair_meet_lte_meet_right(c, d, b)
        pair_lte(pair_meet(b, c), pair_meet(b, d))
        pair_lte_trans(pair_meet(a, c), pair_meet(b, c), pair_meet(b, d))
        pair_lte(pair_meet(a, c), pair_meet(b, d))
    }
}

/// Pair join is jointly monotone in both arguments.
theorem pair_join_lte_join[A: JoinSemilattice, B: JoinSemilattice](
    a: Pair[A, B], b: Pair[A, B], c: Pair[A, B], d: Pair[A, B]) {
    pair_lte(a, b) and pair_lte(c, d) implies pair_lte(pair_join(a, c), pair_join(b, d))
} by {
    if pair_lte(a, b) and pair_lte(c, d) {
        pair_join_lte_join_left(a, b, c)
        pair_lte(pair_join(a, c), pair_join(b, c))
        pair_join_lte_join_right(c, d, b)
        pair_lte(pair_join(b, c), pair_join(b, d))
        pair_lte_trans(pair_join(a, c), pair_join(b, c), pair_join(b, d))
        pair_lte(pair_join(a, c), pair_join(b, d))
    }
}

/// Pair meet absorbs pair join on the right.
theorem pair_meet_absorb_join[A: Lattice, B: Lattice](p: Pair[A, B], q: Pair[A, B]) {
    pair_meet(p, pair_join(p, q)) = p
} by {
    pair_join_first(p, q)
    pair_join_second(p, q)
    pair_join(p, q).first = p.first.join(q.first)
    pair_join(p, q).second = p.second.join(q.second)
    pair_meet_first(p, pair_join(p, q))
    pair_meet_second(p, pair_join(p, q))
    pair_meet(p, pair_join(p, q)).first = p.first.meet(pair_join(p, q).first)
    pair_meet(p, pair_join(p, q)).second = p.second.meet(pair_join(p, q).second)
    pair_meet(p, pair_join(p, q)).first = p.first.meet(p.first.join(q.first))
    pair_meet(p, pair_join(p, q)).second = p.second.meet(p.second.join(q.second))
    meet_absorb_join(p.first, q.first)
    meet_absorb_join(p.second, q.second)
    p.first.meet(p.first.join(q.first)) = p.first
    p.second.meet(p.second.join(q.second)) = p.second
    pair_meet(p, pair_join(p, q)).first = p.first
    pair_meet(p, pair_join(p, q)).second = p.second
    pair_ext(pair_meet(p, pair_join(p, q)), p)
}

/// Pair join absorbs pair meet on the right.
theorem pair_join_absorb_meet[A: Lattice, B: Lattice](p: Pair[A, B], q: Pair[A, B]) {
    pair_join(p, pair_meet(p, q)) = p
} by {
    pair_meet_first(p, q)
    pair_meet_second(p, q)
    pair_meet(p, q).first = p.first.meet(q.first)
    pair_meet(p, q).second = p.second.meet(q.second)
    pair_join_first(p, pair_meet(p, q))
    pair_join_second(p, pair_meet(p, q))
    pair_join(p, pair_meet(p, q)).first = p.first.join(pair_meet(p, q).first)
    pair_join(p, pair_meet(p, q)).second = p.second.join(pair_meet(p, q).second)
    pair_join(p, pair_meet(p, q)).first = p.first.join(p.first.meet(q.first))
    pair_join(p, pair_meet(p, q)).second = p.second.join(p.second.meet(q.second))
    join_absorb_meet(p.first, q.first)
    join_absorb_meet(p.second, q.second)
    p.first.join(p.first.meet(q.first)) = p.first
    p.second.join(p.second.meet(q.second)) = p.second
    pair_join(p, pair_meet(p, q)).first = p.first
    pair_join(p, pair_meet(p, q)).second = p.second
    pair_ext(pair_join(p, pair_meet(p, q)), p)
}

/// Pair meet absorbs pair join on the left.
theorem pair_meet_absorb_join_left[A: Lattice, B: Lattice](p: Pair[A, B], q: Pair[A, B]) {
    pair_meet(pair_join(p, q), p) = p
} by {
    pair_meet_comm(pair_join(p, q), p)
    pair_meet_absorb_join(p, q)
}

/// Pair join absorbs pair meet on the left.
theorem pair_join_absorb_meet_left[A: Lattice, B: Lattice](p: Pair[A, B], q: Pair[A, B]) {
    pair_join(pair_meet(p, q), p) = p
} by {
    pair_join_comm(pair_meet(p, q), p)
    pair_join_absorb_meet(p, q)
}

/// Pair meet distributes over pair join on the left.
theorem pair_meet_join_distrib_left[A: DistribLattice, B: DistribLattice](
    p: Pair[A, B], q: Pair[A, B], r: Pair[A, B]
) {
    pair_meet(p, pair_join(q, r)) = pair_join(pair_meet(p, q), pair_meet(p, r))
} by {
    pair_join_first(q, r)
    pair_join_second(q, r)
    pair_meet_first(p, pair_join(q, r))
    pair_meet_second(p, pair_join(q, r))
    pair_meet(p, pair_join(q, r)).first = p.first.meet(q.first.join(r.first))
    pair_meet(p, pair_join(q, r)).second = p.second.meet(q.second.join(r.second))
    meet_join_distrib_left(p.first, q.first, r.first)
    meet_join_distrib_left(p.second, q.second, r.second)
    pair_meet(p, pair_join(q, r)).first = p.first.meet(q.first).join(p.first.meet(r.first))
    pair_meet(p, pair_join(q, r)).second = p.second.meet(q.second).join(p.second.meet(r.second))
    pair_meet_first(p, q)
    pair_meet_second(p, q)
    pair_meet_first(p, r)
    pair_meet_second(p, r)
    pair_join_first(pair_meet(p, q), pair_meet(p, r))
    pair_join_second(pair_meet(p, q), pair_meet(p, r))
    pair_join(pair_meet(p, q), pair_meet(p, r)).first = p.first.meet(q.first).join(p.first.meet(r.first))
    pair_join(pair_meet(p, q), pair_meet(p, r)).second = p.second.meet(q.second).join(p.second.meet(r.second))
    pair_ext(pair_meet(p, pair_join(q, r)), pair_join(pair_meet(p, q), pair_meet(p, r)))
}

/// Pair join distributes over pair meet on the left.
theorem pair_join_meet_distrib_left[A: DistribLattice, B: DistribLattice](
    p: Pair[A, B], q: Pair[A, B], r: Pair[A, B]
) {
    pair_join(p, pair_meet(q, r)) = pair_meet(pair_join(p, q), pair_join(p, r))
} by {
    pair_meet_first(q, r)
    pair_meet_second(q, r)
    pair_join_first(p, pair_meet(q, r))
    pair_join_second(p, pair_meet(q, r))
    pair_join(p, pair_meet(q, r)).first = p.first.join(q.first.meet(r.first))
    pair_join(p, pair_meet(q, r)).second = p.second.join(q.second.meet(r.second))
    join_meet_distrib_left(p.first, q.first, r.first)
    join_meet_distrib_left(p.second, q.second, r.second)
    pair_join(p, pair_meet(q, r)).first = p.first.join(q.first).meet(p.first.join(r.first))
    pair_join(p, pair_meet(q, r)).second = p.second.join(q.second).meet(p.second.join(r.second))
    pair_join_first(p, q)
    pair_join_second(p, q)
    pair_join_first(p, r)
    pair_join_second(p, r)
    pair_meet_first(pair_join(p, q), pair_join(p, r))
    pair_meet_second(pair_join(p, q), pair_join(p, r))
    pair_meet(pair_join(p, q), pair_join(p, r)).first = p.first.join(q.first).meet(p.first.join(r.first))
    pair_meet(pair_join(p, q), pair_join(p, r)).second = p.second.join(q.second).meet(p.second.join(r.second))
    pair_ext(pair_join(p, pair_meet(q, r)), pair_meet(pair_join(p, q), pair_join(p, r)))
}

/// Pair meet distributes over pair join on the right.
theorem pair_meet_join_distrib_right[A: DistribLattice, B: DistribLattice](
    p: Pair[A, B], q: Pair[A, B], r: Pair[A, B]
) {
    pair_meet(pair_join(p, q), r) = pair_join(pair_meet(p, r), pair_meet(q, r))
} by {
    pair_meet_comm(pair_join(p, q), r)
    pair_meet_join_distrib_left(r, p, q)
    pair_meet_comm(r, p)
    pair_meet_comm(r, q)
}

/// Pair join distributes over pair meet on the right.
theorem pair_join_meet_distrib_right[A: DistribLattice, B: DistribLattice](
    p: Pair[A, B], q: Pair[A, B], r: Pair[A, B]
) {
    pair_join(pair_meet(p, q), r) = pair_meet(pair_join(p, r), pair_join(q, r))
} by {
    pair_join_comm(pair_meet(p, q), r)
    pair_join_meet_distrib_left(r, p, q)
    pair_join_comm(r, p)
    pair_join_comm(r, q)
}

/// Swapping commutes with componentwise pair meet.
theorem pair_swap_meet[A: MeetSemilattice, B: MeetSemilattice](p: Pair[A, B], q: Pair[A, B]) {
    pair_meet(p, q).swap = pair_meet(p.swap, q.swap)
} by {
    pair_meet_first(p, q)
    pair_meet_second(p, q)
    swap_first(pair_meet(p, q))
    swap_second(pair_meet(p, q))
    pair_meet(p, q).swap.first = p.second.meet(q.second)
    pair_meet(p, q).swap.second = p.first.meet(q.first)
    swap_first(p)
    swap_second(p)
    swap_first(q)
    swap_second(q)
    pair_meet_first(p.swap, q.swap)
    pair_meet_second(p.swap, q.swap)
    pair_meet(p.swap, q.swap).first = p.second.meet(q.second)
    pair_meet(p.swap, q.swap).second = p.first.meet(q.first)
    pair_ext(pair_meet(p, q).swap, pair_meet(p.swap, q.swap))
}

/// Swapping commutes with componentwise pair join.
theorem pair_swap_join[A: JoinSemilattice, B: JoinSemilattice](p: Pair[A, B], q: Pair[A, B]) {
    pair_join(p, q).swap = pair_join(p.swap, q.swap)
} by {
    pair_join_first(p, q)
    pair_join_second(p, q)
    swap_first(pair_join(p, q))
    swap_second(pair_join(p, q))
    pair_join(p, q).swap.first = p.second.join(q.second)
    pair_join(p, q).swap.second = p.first.join(q.first)
    swap_first(p)
    swap_second(p)
    swap_first(q)
    swap_second(q)
    pair_join_first(p.swap, q.swap)
    pair_join_second(p.swap, q.swap)
    pair_join(p.swap, q.swap).first = p.second.join(q.second)
    pair_join(p.swap, q.swap).second = p.first.join(q.first)
    pair_ext(pair_join(p, q).swap, pair_join(p.swap, q.swap))
}

/// The componentwise meet of two diagonal pairs is the diagonal pair of the meet.
theorem pair_meet_diag[A: MeetSemilattice](a: A, b: A) {
    pair_meet(Pair.new(a, a), Pair.new(b, b)) = Pair.new(a.meet(b), a.meet(b))
} by {
    pair_meet_first(Pair.new(a, a), Pair.new(b, b))
    pair_meet_second(Pair.new(a, a), Pair.new(b, b))
    pair_new_first(a, a)
    pair_new_second(a, a)
    pair_new_first(b, b)
    pair_new_second(b, b)
    pair_meet(Pair.new(a, a), Pair.new(b, b)).first = a.meet(b)
    pair_meet(Pair.new(a, a), Pair.new(b, b)).second = a.meet(b)
    pair_new_first(a.meet(b), a.meet(b))
    pair_new_second(a.meet(b), a.meet(b))
    Pair.new(a.meet(b), a.meet(b)).first = a.meet(b)
    Pair.new(a.meet(b), a.meet(b)).second = a.meet(b)
    pair_ext(pair_meet(Pair.new(a, a), Pair.new(b, b)), Pair.new(a.meet(b), a.meet(b)))
}

/// The componentwise join of two diagonal pairs is the diagonal pair of the join.
theorem pair_join_diag[A: JoinSemilattice](a: A, b: A) {
    pair_join(Pair.new(a, a), Pair.new(b, b)) = Pair.new(a.join(b), a.join(b))
} by {
    pair_join_first(Pair.new(a, a), Pair.new(b, b))
    pair_join_second(Pair.new(a, a), Pair.new(b, b))
    pair_new_first(a, a)
    pair_new_second(a, a)
    pair_new_first(b, b)
    pair_new_second(b, b)
    pair_join(Pair.new(a, a), Pair.new(b, b)).first = a.join(b)
    pair_join(Pair.new(a, a), Pair.new(b, b)).second = a.join(b)
    pair_new_first(a.join(b), a.join(b))
    pair_new_second(a.join(b), a.join(b))
    Pair.new(a.join(b), a.join(b)).first = a.join(b)
    Pair.new(a.join(b), a.join(b)).second = a.join(b)
    pair_ext(pair_join(Pair.new(a, a), Pair.new(b, b)), Pair.new(a.join(b), a.join(b)))
}

/// Componentwise pair meet equals the swap of the swapped pair meet.
theorem pair_meet_eq_swap_meet_swap[A: MeetSemilattice, B: MeetSemilattice](p: Pair[A, B], q: Pair[A, B]) {
    pair_meet(p, q) = pair_meet(p.swap, q.swap).swap
} by {
    pair_swap_meet(p, q)
    pair_meet(p, q).swap = pair_meet(p.swap, q.swap)
    swap_swap(pair_meet(p, q))
    pair_meet(p, q).swap.swap = pair_meet(p, q)
    pair_meet(p.swap, q.swap).swap = pair_meet(p, q)
}

/// Componentwise pair join equals the swap of the swapped pair join.
theorem pair_join_eq_swap_join_swap[A: JoinSemilattice, B: JoinSemilattice](p: Pair[A, B], q: Pair[A, B]) {
    pair_join(p, q) = pair_join(p.swap, q.swap).swap
} by {
    pair_swap_join(p, q)
    pair_join(p, q).swap = pair_join(p.swap, q.swap)
    swap_swap(pair_join(p, q))
    pair_join(p, q).swap.swap = pair_join(p, q)
    pair_join(p.swap, q.swap).swap = pair_join(p, q)
}

/// The componentwise meet of two `Pair.new` pairs is `Pair.new` of the componentwise meets.
theorem pair_meet_new[A: MeetSemilattice, B: MeetSemilattice](a: A, b: B, c: A, d: B) {
    pair_meet(Pair.new(a, b), Pair.new(c, d)) = Pair.new(a.meet(c), b.meet(d))
} by {
    pair_new_first(a, b)
    pair_new_second(a, b)
    pair_new_first(c, d)
    pair_new_second(c, d)
    pair_meet_first(Pair.new(a, b), Pair.new(c, d))
    pair_meet_second(Pair.new(a, b), Pair.new(c, d))
    pair_meet(Pair.new(a, b), Pair.new(c, d)).first = a.meet(c)
    pair_meet(Pair.new(a, b), Pair.new(c, d)).second = b.meet(d)
    pair_new_first(a.meet(c), b.meet(d))
    pair_new_second(a.meet(c), b.meet(d))
    Pair.new(a.meet(c), b.meet(d)).first = a.meet(c)
    Pair.new(a.meet(c), b.meet(d)).second = b.meet(d)
    pair_ext(pair_meet(Pair.new(a, b), Pair.new(c, d)), Pair.new(a.meet(c), b.meet(d)))
}

/// The componentwise join of two `Pair.new` pairs is `Pair.new` of the componentwise joins.
theorem pair_join_new[A: JoinSemilattice, B: JoinSemilattice](a: A, b: B, c: A, d: B) {
    pair_join(Pair.new(a, b), Pair.new(c, d)) = Pair.new(a.join(c), b.join(d))
} by {
    pair_new_first(a, b)
    pair_new_second(a, b)
    pair_new_first(c, d)
    pair_new_second(c, d)
    pair_join_first(Pair.new(a, b), Pair.new(c, d))
    pair_join_second(Pair.new(a, b), Pair.new(c, d))
    pair_join(Pair.new(a, b), Pair.new(c, d)).first = a.join(c)
    pair_join(Pair.new(a, b), Pair.new(c, d)).second = b.join(d)
    pair_new_first(a.join(c), b.join(d))
    pair_new_second(a.join(c), b.join(d))
    Pair.new(a.join(c), b.join(d)).first = a.join(c)
    Pair.new(a.join(c), b.join(d)).second = b.join(d)
    pair_ext(pair_join(Pair.new(a, b), Pair.new(c, d)), Pair.new(a.join(c), b.join(d)))
}

/// Pair meet is associative in the reversed nesting.
theorem pair_meet_assoc_rev[A: MeetSemilattice, B: MeetSemilattice](p: Pair[A, B], q: Pair[A, B], r: Pair[A, B]) {
    pair_meet(p, pair_meet(q, r)) = pair_meet(pair_meet(p, q), r)
} by {
    pair_meet_assoc(p, q, r)
}

/// Pair join is associative in the reversed nesting.
theorem pair_join_assoc_rev[A: JoinSemilattice, B: JoinSemilattice](p: Pair[A, B], q: Pair[A, B], r: Pair[A, B]) {
    pair_join(p, pair_join(q, r)) = pair_join(pair_join(p, q), r)
} by {
    pair_join_assoc(p, q, r)
}

/// The outer left argument of a nested pair meet may be exchanged.
theorem pair_meet_left_comm[A: MeetSemilattice, B: MeetSemilattice](p: Pair[A, B], q: Pair[A, B], r: Pair[A, B]) {
    pair_meet(p, pair_meet(q, r)) = pair_meet(q, pair_meet(p, r))
} by {
    pair_meet_assoc_rev(p, q, r)
    pair_meet_comm(p, q)
    pair_meet_assoc(q, p, r)
}

/// The outer left argument of a nested pair join may be exchanged.
theorem pair_join_left_comm[A: JoinSemilattice, B: JoinSemilattice](p: Pair[A, B], q: Pair[A, B], r: Pair[A, B]) {
    pair_join(p, pair_join(q, r)) = pair_join(q, pair_join(p, r))
} by {
    pair_join_assoc_rev(p, q, r)
    pair_join_comm(p, q)
    pair_join_assoc(q, p, r)
}

/// The two right arguments of an iterated pair meet may be exchanged.
theorem pair_meet_right_comm[A: MeetSemilattice, B: MeetSemilattice](p: Pair[A, B], q: Pair[A, B], r: Pair[A, B]) {
    pair_meet(pair_meet(p, q), r) = pair_meet(pair_meet(p, r), q)
} by {
    pair_meet_assoc(p, q, r)
    pair_meet_comm(q, r)
    pair_meet_assoc_rev(p, r, q)
}

/// The two right arguments of an iterated pair join may be exchanged.
theorem pair_join_right_comm[A: JoinSemilattice, B: JoinSemilattice](p: Pair[A, B], q: Pair[A, B], r: Pair[A, B]) {
    pair_join(pair_join(p, q), r) = pair_join(pair_join(p, r), q)
} by {
    pair_join_assoc(p, q, r)
    pair_join_comm(q, r)
    pair_join_assoc_rev(p, r, q)
}

/// Swapping the left argument of pair meet equals swapping the whole result after swapping the right argument.
theorem pair_swap_meet_left[A: MeetSemilattice, B: MeetSemilattice](p: Pair[A, B], q: Pair[B, A]) {
    pair_meet(p.swap, q) = pair_meet(p, q.swap).swap
} by {
    swap_swap(q)
    pair_swap_meet(p, q.swap)
    pair_meet(p, q.swap).swap = pair_meet(p.swap, q.swap.swap)
    pair_meet(p, q.swap).swap = pair_meet(p.swap, q)
}

/// Swapping the right argument of pair meet equals swapping the whole result after swapping the left argument.
theorem pair_swap_meet_right[A: MeetSemilattice, B: MeetSemilattice](p: Pair[B, A], q: Pair[A, B]) {
    pair_meet(p, q.swap) = pair_meet(p.swap, q).swap
} by {
    swap_swap(p)
    pair_swap_meet(p.swap, q)
    pair_meet(p.swap, q).swap = pair_meet(p.swap.swap, q.swap)
    pair_meet(p.swap, q).swap = pair_meet(p, q.swap)
}

/// Swapping the left argument of pair join equals swapping the whole result after swapping the right argument.
theorem pair_swap_join_left[A: JoinSemilattice, B: JoinSemilattice](p: Pair[A, B], q: Pair[B, A]) {
    pair_join(p.swap, q) = pair_join(p, q.swap).swap
} by {
    swap_swap(q)
    pair_swap_join(p, q.swap)
    pair_join(p, q.swap).swap = pair_join(p.swap, q.swap.swap)
    pair_join(p, q.swap).swap = pair_join(p.swap, q)
}

/// Swapping the right argument of pair join equals swapping the whole result after swapping the left argument.
theorem pair_swap_join_right[A: JoinSemilattice, B: JoinSemilattice](p: Pair[B, A], q: Pair[A, B]) {
    pair_join(p, q.swap) = pair_join(p.swap, q).swap
} by {
    swap_swap(p)
    pair_swap_join(p.swap, q)
    pair_join(p.swap, q).swap = pair_join(p.swap.swap, q.swap)
    pair_join(p.swap, q).swap = pair_join(p, q.swap)
}

/// The componentwise meet of a diagonal pair with an arbitrary pair on the left.
theorem pair_meet_diag_left[A: MeetSemilattice](a: A, q: Pair[A, A]) {
    pair_meet(Pair.new(a, a), q) = Pair.new(a.meet(q.first), a.meet(q.second))
} by {
    pair_meet_first(Pair.new(a, a), q)
    pair_meet_second(Pair.new(a, a), q)
    pair_new_first(a, a)
    pair_new_second(a, a)
    pair_meet(Pair.new(a, a), q).first = a.meet(q.first)
    pair_meet(Pair.new(a, a), q).second = a.meet(q.second)
    pair_new_first(a.meet(q.first), a.meet(q.second))
    pair_new_second(a.meet(q.first), a.meet(q.second))
    Pair.new(a.meet(q.first), a.meet(q.second)).first = a.meet(q.first)
    Pair.new(a.meet(q.first), a.meet(q.second)).second = a.meet(q.second)
    pair_ext(pair_meet(Pair.new(a, a), q), Pair.new(a.meet(q.first), a.meet(q.second)))
}

/// The componentwise meet of an arbitrary pair with a diagonal pair on the right.
theorem pair_meet_diag_right[A: MeetSemilattice](p: Pair[A, A], a: A) {
    pair_meet(p, Pair.new(a, a)) = Pair.new(p.first.meet(a), p.second.meet(a))
} by {
    pair_meet_first(p, Pair.new(a, a))
    pair_meet_second(p, Pair.new(a, a))
    pair_new_first(a, a)
    pair_new_second(a, a)
    pair_meet(p, Pair.new(a, a)).first = p.first.meet(a)
    pair_meet(p, Pair.new(a, a)).second = p.second.meet(a)
    pair_new_first(p.first.meet(a), p.second.meet(a))
    pair_new_second(p.first.meet(a), p.second.meet(a))
    Pair.new(p.first.meet(a), p.second.meet(a)).first = p.first.meet(a)
    Pair.new(p.first.meet(a), p.second.meet(a)).second = p.second.meet(a)
    pair_ext(pair_meet(p, Pair.new(a, a)), Pair.new(p.first.meet(a), p.second.meet(a)))
}

/// The componentwise join of a diagonal pair with an arbitrary pair on the left.
theorem pair_join_diag_left[A: JoinSemilattice](a: A, q: Pair[A, A]) {
    pair_join(Pair.new(a, a), q) = Pair.new(a.join(q.first), a.join(q.second))
} by {
    pair_join_first(Pair.new(a, a), q)
    pair_join_second(Pair.new(a, a), q)
    pair_new_first(a, a)
    pair_new_second(a, a)
    pair_join(Pair.new(a, a), q).first = a.join(q.first)
    pair_join(Pair.new(a, a), q).second = a.join(q.second)
    pair_new_first(a.join(q.first), a.join(q.second))
    pair_new_second(a.join(q.first), a.join(q.second))
    Pair.new(a.join(q.first), a.join(q.second)).first = a.join(q.first)
    Pair.new(a.join(q.first), a.join(q.second)).second = a.join(q.second)
    pair_ext(pair_join(Pair.new(a, a), q), Pair.new(a.join(q.first), a.join(q.second)))
}

/// The componentwise join of an arbitrary pair with a diagonal pair on the right.
theorem pair_join_diag_right[A: JoinSemilattice](p: Pair[A, A], a: A) {
    pair_join(p, Pair.new(a, a)) = Pair.new(p.first.join(a), p.second.join(a))
} by {
    pair_join_first(p, Pair.new(a, a))
    pair_join_second(p, Pair.new(a, a))
    pair_new_first(a, a)
    pair_new_second(a, a)
    pair_join(p, Pair.new(a, a)).first = p.first.join(a)
    pair_join(p, Pair.new(a, a)).second = p.second.join(a)
    pair_new_first(p.first.join(a), p.second.join(a))
    pair_new_second(p.first.join(a), p.second.join(a))
    Pair.new(p.first.join(a), p.second.join(a)).first = p.first.join(a)
    Pair.new(p.first.join(a), p.second.join(a)).second = p.second.join(a)
    pair_ext(pair_join(p, Pair.new(a, a)), Pair.new(p.first.join(a), p.second.join(a)))
}

/// The first component of the componentwise meet of two `Pair.new` pairs.
theorem pair_meet_new_first[A: MeetSemilattice, B: MeetSemilattice](a: A, b: B, c: A, d: B) {
    pair_meet(Pair.new(a, b), Pair.new(c, d)).first = a.meet(c)
} by {
    pair_meet_new(a, b, c, d)
    pair_new_first(a.meet(c), b.meet(d))
}

/// The second component of the componentwise meet of two `Pair.new` pairs.
theorem pair_meet_new_second[A: MeetSemilattice, B: MeetSemilattice](a: A, b: B, c: A, d: B) {
    pair_meet(Pair.new(a, b), Pair.new(c, d)).second = b.meet(d)
} by {
    pair_meet_new(a, b, c, d)
    pair_new_second(a.meet(c), b.meet(d))
}

/// The first component of the componentwise join of two `Pair.new` pairs.
theorem pair_join_new_first[A: JoinSemilattice, B: JoinSemilattice](a: A, b: B, c: A, d: B) {
    pair_join(Pair.new(a, b), Pair.new(c, d)).first = a.join(c)
} by {
    pair_join_new(a, b, c, d)
    pair_new_first(a.join(c), b.join(d))
}

/// The second component of the componentwise join of two `Pair.new` pairs.
theorem pair_join_new_second[A: JoinSemilattice, B: JoinSemilattice](a: A, b: B, c: A, d: B) {
    pair_join(Pair.new(a, b), Pair.new(c, d)).second = b.join(d)
} by {
    pair_join_new(a, b, c, d)
    pair_new_second(a.join(c), b.join(d))
}

/// The first component of the componentwise meet of two diagonal pairs.
theorem pair_meet_diag_first[A: MeetSemilattice](a: A, b: A) {
    pair_meet(Pair.new(a, a), Pair.new(b, b)).first = a.meet(b)
} by {
    pair_meet_diag(a, b)
    pair_new_first(a.meet(b), a.meet(b))
}

/// The second component of the componentwise meet of two diagonal pairs.
theorem pair_meet_diag_second[A: MeetSemilattice](a: A, b: A) {
    pair_meet(Pair.new(a, a), Pair.new(b, b)).second = a.meet(b)
} by {
    pair_meet_diag(a, b)
    pair_new_second(a.meet(b), a.meet(b))
}

/// The first component of the componentwise join of two diagonal pairs.
theorem pair_join_diag_first[A: JoinSemilattice](a: A, b: A) {
    pair_join(Pair.new(a, a), Pair.new(b, b)).first = a.join(b)
} by {
    pair_join_diag(a, b)
    pair_new_first(a.join(b), a.join(b))
}

/// The second component of the componentwise join of two diagonal pairs.
theorem pair_join_diag_second[A: JoinSemilattice](a: A, b: A) {
    pair_join(Pair.new(a, a), Pair.new(b, b)).second = a.join(b)
} by {
    pair_join_diag(a, b)
    pair_new_second(a.join(b), a.join(b))
}

/// A pair meet is strictly below its left argument iff the left is not below the right.
theorem pair_meet_lt_left_iff_not_lte[A: MeetSemilattice, B: MeetSemilattice](p: Pair[A, B], q: Pair[A, B]) {
    pair_lt(pair_meet(p, q), p) = not pair_lte(p, q)
} by {
    pair_meet_lte_left(p, q)
    pair_meet_eq_left_iff_lte(p, q)
    if pair_lt(pair_meet(p, q), p) {
        pair_lt_imp_ne(pair_meet(p, q), p)
        pair_meet(p, q) != p
        not pair_lte(p, q)
    }
    if not pair_lte(p, q) {
        pair_meet(p, q) != p
        pair_lt_of_lte_of_ne(pair_meet(p, q), p)
        pair_lt(pair_meet(p, q), p)
    }
}

/// A pair meet is strictly below its right argument iff the right is not below the left.
theorem pair_meet_lt_right_iff_not_lte[A: MeetSemilattice, B: MeetSemilattice](p: Pair[A, B], q: Pair[A, B]) {
    pair_lt(pair_meet(p, q), q) = not pair_lte(q, p)
} by {
    pair_meet_lte_right(p, q)
    pair_meet_eq_right_iff_lte(p, q)
    if pair_lt(pair_meet(p, q), q) {
        pair_lt_imp_ne(pair_meet(p, q), q)
        pair_meet(p, q) != q
        not pair_lte(q, p)
    }
    if not pair_lte(q, p) {
        pair_meet(p, q) != q
        pair_lt_of_lte_of_ne(pair_meet(p, q), q)
        pair_lt(pair_meet(p, q), q)
    }
}

/// A pair join is strictly above its left argument iff the right is not below the left.
theorem pair_join_lt_left_iff_not_lte[A: JoinSemilattice, B: JoinSemilattice](p: Pair[A, B], q: Pair[A, B]) {
    pair_lt(p, pair_join(p, q)) = not pair_lte(q, p)
} by {
    pair_lte_join_left(p, q)
    pair_join_eq_left_iff_lte(p, q)
    if pair_lt(p, pair_join(p, q)) {
        pair_lt_imp_ne(p, pair_join(p, q))
        pair_join(p, q) != p
        not pair_lte(q, p)
    }
    if not pair_lte(q, p) {
        pair_join(p, q) != p
        pair_lt_of_lte_of_ne(p, pair_join(p, q))
        pair_lt(p, pair_join(p, q))
    }
}

/// A pair join is strictly above its right argument iff the left is not below the right.
theorem pair_join_lt_right_iff_not_lte[A: JoinSemilattice, B: JoinSemilattice](p: Pair[A, B], q: Pair[A, B]) {
    pair_lt(q, pair_join(p, q)) = not pair_lte(p, q)
} by {
    pair_lte_join_right(p, q)
    pair_join_eq_right_iff_lte(p, q)
    if pair_lt(q, pair_join(p, q)) {
        pair_lt_imp_ne(q, pair_join(p, q))
        pair_join(p, q) != q
        not pair_lte(p, q)
    }
    if not pair_lte(p, q) {
        pair_join(p, q) != q
        pair_lt_of_lte_of_ne(q, pair_join(p, q))
        pair_lt(q, pair_join(p, q))
    }
}

/// In a lattice, the componentwise pair meet is always below the componentwise pair join.
theorem pair_lte_meet_join[A: Lattice, B: Lattice](p: Pair[A, B], q: Pair[A, B]) {
    pair_lte(pair_meet(p, q), pair_join(p, q))
} by {
    pair_meet_lte_left(p, q)
    pair_lte(pair_meet(p, q), p)
    pair_lte_join_left(p, q)
    pair_lte(p, pair_join(p, q))
    pair_lte_trans(pair_meet(p, q), p, pair_join(p, q))
}

/// In a lattice, the componentwise pair meet equals the componentwise pair join iff the pairs are equal.
theorem pair_meet_eq_join_iff_eq[A: Lattice, B: Lattice](p: Pair[A, B], q: Pair[A, B]) {
    pair_meet(p, q) = pair_join(p, q) = (p = q)
} by {
    if pair_meet(p, q) = pair_join(p, q) {
        pair_meet_lte_left(p, q)
        pair_lte(pair_meet(p, q), p)
        pair_lte_join_left(p, q)
        pair_lte(p, pair_join(p, q))
        pair_lte(p, pair_meet(p, q))
        pair_lte_antisymm(pair_meet(p, q), p)
        pair_meet(p, q) = p
        pair_meet_eq_left_iff_lte(p, q)
        pair_lte(p, q)
        pair_meet_lte_right(p, q)
        pair_lte(pair_meet(p, q), q)
        pair_lte_join_right(p, q)
        pair_lte(q, pair_join(p, q))
        pair_lte(q, pair_meet(p, q))
        pair_lte_antisymm(pair_meet(p, q), q)
        pair_meet(p, q) = q
        p = q
    }
    if p = q {
        pair_meet_idem(p)
        pair_join_idem(p)
        pair_meet(p, q) = p
        pair_join(p, q) = p
        pair_meet(p, q) = pair_join(p, q)
    }
}

/// In a lattice, the componentwise pair meet is strictly below the componentwise pair join iff the pairs are distinct.
theorem pair_meet_lt_join_iff_ne[A: Lattice, B: Lattice](p: Pair[A, B], q: Pair[A, B]) {
    pair_lt(pair_meet(p, q), pair_join(p, q)) = (p != q)
} by {
    pair_lte_meet_join(p, q)
    pair_meet_eq_join_iff_eq(p, q)
    if pair_lt(pair_meet(p, q), pair_join(p, q)) {
        pair_lt_imp_ne(pair_meet(p, q), pair_join(p, q))
        pair_meet(p, q) != pair_join(p, q)
        p != q
    }
    if p != q {
        pair_meet(p, q) != pair_join(p, q)
        pair_lt_of_lte_of_ne(pair_meet(p, q), pair_join(p, q))
        pair_lt(pair_meet(p, q), pair_join(p, q))
    }
}

/// A pair meet is strictly below its left argument when the left is not below the right.
theorem pair_meet_lt_left_of_not_lte[A: MeetSemilattice, B: MeetSemilattice](p: Pair[A, B], q: Pair[A, B]) {
    not pair_lte(p, q) implies pair_lt(pair_meet(p, q), p)
} by {
    pair_meet_lt_left_iff_not_lte(p, q)
}

/// A pair meet is strictly below its right argument when the right is not below the left.
theorem pair_meet_lt_right_of_not_lte[A: MeetSemilattice, B: MeetSemilattice](p: Pair[A, B], q: Pair[A, B]) {
    not pair_lte(q, p) implies pair_lt(pair_meet(p, q), q)
} by {
    pair_meet_lt_right_iff_not_lte(p, q)
}

/// A pair join is strictly above its left argument when the right is not below the left.
theorem pair_join_lt_left_of_not_lte[A: JoinSemilattice, B: JoinSemilattice](p: Pair[A, B], q: Pair[A, B]) {
    not pair_lte(q, p) implies pair_lt(p, pair_join(p, q))
} by {
    pair_join_lt_left_iff_not_lte(p, q)
}

/// A pair join is strictly above its right argument when the left is not below the right.
theorem pair_join_lt_right_of_not_lte[A: JoinSemilattice, B: JoinSemilattice](p: Pair[A, B], q: Pair[A, B]) {
    not pair_lte(p, q) implies pair_lt(q, pair_join(p, q))
} by {
    pair_join_lt_right_iff_not_lte(p, q)
}

/// First component of the componentwise meet with a diagonal pair on the left.
theorem pair_meet_diag_left_first[A: MeetSemilattice](a: A, q: Pair[A, A]) {
    pair_meet(Pair.new(a, a), q).first = a.meet(q.first)
} by {
    pair_meet_diag_left(a, q)
    pair_new_first(a.meet(q.first), a.meet(q.second))
}

/// Second component of the componentwise meet with a diagonal pair on the left.
theorem pair_meet_diag_left_second[A: MeetSemilattice](a: A, q: Pair[A, A]) {
    pair_meet(Pair.new(a, a), q).second = a.meet(q.second)
} by {
    pair_meet_diag_left(a, q)
    pair_new_second(a.meet(q.first), a.meet(q.second))
}

/// First component of the componentwise meet with a diagonal pair on the right.
theorem pair_meet_diag_right_first[A: MeetSemilattice](p: Pair[A, A], a: A) {
    pair_meet(p, Pair.new(a, a)).first = p.first.meet(a)
} by {
    pair_meet_diag_right(p, a)
    pair_new_first(p.first.meet(a), p.second.meet(a))
}

/// Second component of the componentwise meet with a diagonal pair on the right.
theorem pair_meet_diag_right_second[A: MeetSemilattice](p: Pair[A, A], a: A) {
    pair_meet(p, Pair.new(a, a)).second = p.second.meet(a)
} by {
    pair_meet_diag_right(p, a)
    pair_new_second(p.first.meet(a), p.second.meet(a))
}

/// First component of the componentwise join with a diagonal pair on the left.
theorem pair_join_diag_left_first[A: JoinSemilattice](a: A, q: Pair[A, A]) {
    pair_join(Pair.new(a, a), q).first = a.join(q.first)
} by {
    pair_join_diag_left(a, q)
    pair_new_first(a.join(q.first), a.join(q.second))
}

/// Second component of the componentwise join with a diagonal pair on the left.
theorem pair_join_diag_left_second[A: JoinSemilattice](a: A, q: Pair[A, A]) {
    pair_join(Pair.new(a, a), q).second = a.join(q.second)
} by {
    pair_join_diag_left(a, q)
    pair_new_second(a.join(q.first), a.join(q.second))
}

/// First component of the componentwise join with a diagonal pair on the right.
theorem pair_join_diag_right_first[A: JoinSemilattice](p: Pair[A, A], a: A) {
    pair_join(p, Pair.new(a, a)).first = p.first.join(a)
} by {
    pair_join_diag_right(p, a)
    pair_new_first(p.first.join(a), p.second.join(a))
}

/// Second component of the componentwise join with a diagonal pair on the right.
theorem pair_join_diag_right_second[A: JoinSemilattice](p: Pair[A, A], a: A) {
    pair_join(p, Pair.new(a, a)).second = p.second.join(a)
} by {
    pair_join_diag_right(p, a)
    pair_new_second(p.first.join(a), p.second.join(a))
}

/// First component of the swap of a componentwise pair meet.
theorem pair_meet_swap_first[A: MeetSemilattice, B: MeetSemilattice](p: Pair[A, B], q: Pair[A, B]) {
    pair_meet(p, q).swap.first = p.second.meet(q.second)
} by {
    swap_first(pair_meet(p, q))
    pair_meet_second(p, q)
}

/// Second component of the swap of a componentwise pair meet.
theorem pair_meet_swap_second[A: MeetSemilattice, B: MeetSemilattice](p: Pair[A, B], q: Pair[A, B]) {
    pair_meet(p, q).swap.second = p.first.meet(q.first)
} by {
    swap_second(pair_meet(p, q))
    pair_meet_first(p, q)
}

/// First component of the swap of a componentwise pair join.
theorem pair_join_swap_first[A: JoinSemilattice, B: JoinSemilattice](p: Pair[A, B], q: Pair[A, B]) {
    pair_join(p, q).swap.first = p.second.join(q.second)
} by {
    swap_first(pair_join(p, q))
    pair_join_second(p, q)
}

/// Second component of the swap of a componentwise pair join.
theorem pair_join_swap_second[A: JoinSemilattice, B: JoinSemilattice](p: Pair[A, B], q: Pair[A, B]) {
    pair_join(p, q).swap.second = p.first.join(q.first)
} by {
    swap_second(pair_join(p, q))
    pair_join_first(p, q)
}

/// First component of a left-nested triple pair meet is the meet chain of first components.
theorem pair_meet_assoc_first[A: MeetSemilattice, B: MeetSemilattice](p: Pair[A, B], q: Pair[A, B], r: Pair[A, B]) {
    pair_meet(pair_meet(p, q), r).first = p.first.meet(q.first).meet(r.first)
} by {
    pair_meet_first(pair_meet(p, q), r)
    pair_meet_first(p, q)
}

/// Second component of a left-nested triple pair meet is the meet chain of second components.
theorem pair_meet_assoc_second[A: MeetSemilattice, B: MeetSemilattice](p: Pair[A, B], q: Pair[A, B], r: Pair[A, B]) {
    pair_meet(pair_meet(p, q), r).second = p.second.meet(q.second).meet(r.second)
} by {
    pair_meet_second(pair_meet(p, q), r)
    pair_meet_second(p, q)
}

/// First component of a left-nested triple pair join is the join chain of first components.
theorem pair_join_assoc_first[A: JoinSemilattice, B: JoinSemilattice](p: Pair[A, B], q: Pair[A, B], r: Pair[A, B]) {
    pair_join(pair_join(p, q), r).first = p.first.join(q.first).join(r.first)
} by {
    pair_join_first(pair_join(p, q), r)
    pair_join_first(p, q)
}

/// Second component of a left-nested triple pair join is the join chain of second components.
theorem pair_join_assoc_second[A: JoinSemilattice, B: JoinSemilattice](p: Pair[A, B], q: Pair[A, B], r: Pair[A, B]) {
    pair_join(pair_join(p, q), r).second = p.second.join(q.second).join(r.second)
} by {
    pair_join_second(pair_join(p, q), r)
    pair_join_second(p, q)
}

/// First component of a right-nested triple pair meet is the right-nested meet of first components.
theorem pair_meet_assoc_rev_first[A: MeetSemilattice, B: MeetSemilattice](p: Pair[A, B], q: Pair[A, B], r: Pair[A, B]) {
    pair_meet(p, pair_meet(q, r)).first = p.first.meet(q.first.meet(r.first))
} by {
    pair_meet_first(p, pair_meet(q, r))
    pair_meet_first(q, r)
}

/// Second component of a right-nested triple pair meet is the right-nested meet of second components.
theorem pair_meet_assoc_rev_second[A: MeetSemilattice, B: MeetSemilattice](p: Pair[A, B], q: Pair[A, B], r: Pair[A, B]) {
    pair_meet(p, pair_meet(q, r)).second = p.second.meet(q.second.meet(r.second))
} by {
    pair_meet_second(p, pair_meet(q, r))
    pair_meet_second(q, r)
}

/// First component of a right-nested triple pair join is the right-nested join of first components.
theorem pair_join_assoc_rev_first[A: JoinSemilattice, B: JoinSemilattice](p: Pair[A, B], q: Pair[A, B], r: Pair[A, B]) {
    pair_join(p, pair_join(q, r)).first = p.first.join(q.first.join(r.first))
} by {
    pair_join_first(p, pair_join(q, r))
    pair_join_first(q, r)
}

/// Second component of a right-nested triple pair join is the right-nested join of second components.
theorem pair_join_assoc_rev_second[A: JoinSemilattice, B: JoinSemilattice](p: Pair[A, B], q: Pair[A, B], r: Pair[A, B]) {
    pair_join(p, pair_join(q, r)).second = p.second.join(q.second.join(r.second))
} by {
    pair_join_second(p, pair_join(q, r))
    pair_join_second(q, r)
}

/// The componentwise meet of a `Pair.new` pair with an arbitrary pair on the left.
theorem pair_meet_new_left[A: MeetSemilattice, B: MeetSemilattice](a: A, b: B, q: Pair[A, B]) {
    pair_meet(Pair.new(a, b), q) = Pair.new(a.meet(q.first), b.meet(q.second))
} by {
    pair_meet_first(Pair.new(a, b), q)
    pair_meet_second(Pair.new(a, b), q)
    pair_new_first(a, b)
    pair_new_second(a, b)
    pair_meet(Pair.new(a, b), q).first = a.meet(q.first)
    pair_meet(Pair.new(a, b), q).second = b.meet(q.second)
    pair_new_first(a.meet(q.first), b.meet(q.second))
    pair_new_second(a.meet(q.first), b.meet(q.second))
    Pair.new(a.meet(q.first), b.meet(q.second)).first = a.meet(q.first)
    Pair.new(a.meet(q.first), b.meet(q.second)).second = b.meet(q.second)
    pair_ext(pair_meet(Pair.new(a, b), q), Pair.new(a.meet(q.first), b.meet(q.second)))
}

/// The componentwise meet of an arbitrary pair with a `Pair.new` pair on the right.
theorem pair_meet_new_right[A: MeetSemilattice, B: MeetSemilattice](p: Pair[A, B], a: A, b: B) {
    pair_meet(p, Pair.new(a, b)) = Pair.new(p.first.meet(a), p.second.meet(b))
} by {
    pair_meet_first(p, Pair.new(a, b))
    pair_meet_second(p, Pair.new(a, b))
    pair_new_first(a, b)
    pair_new_second(a, b)
    pair_meet(p, Pair.new(a, b)).first = p.first.meet(a)
    pair_meet(p, Pair.new(a, b)).second = p.second.meet(b)
    pair_new_first(p.first.meet(a), p.second.meet(b))
    pair_new_second(p.first.meet(a), p.second.meet(b))
    Pair.new(p.first.meet(a), p.second.meet(b)).first = p.first.meet(a)
    Pair.new(p.first.meet(a), p.second.meet(b)).second = p.second.meet(b)
    pair_ext(pair_meet(p, Pair.new(a, b)), Pair.new(p.first.meet(a), p.second.meet(b)))
}

/// The componentwise join of a `Pair.new` pair with an arbitrary pair on the left.
theorem pair_join_new_left[A: JoinSemilattice, B: JoinSemilattice](a: A, b: B, q: Pair[A, B]) {
    pair_join(Pair.new(a, b), q) = Pair.new(a.join(q.first), b.join(q.second))
} by {
    pair_join_first(Pair.new(a, b), q)
    pair_join_second(Pair.new(a, b), q)
    pair_new_first(a, b)
    pair_new_second(a, b)
    pair_join(Pair.new(a, b), q).first = a.join(q.first)
    pair_join(Pair.new(a, b), q).second = b.join(q.second)
    pair_new_first(a.join(q.first), b.join(q.second))
    pair_new_second(a.join(q.first), b.join(q.second))
    Pair.new(a.join(q.first), b.join(q.second)).first = a.join(q.first)
    Pair.new(a.join(q.first), b.join(q.second)).second = b.join(q.second)
    pair_ext(pair_join(Pair.new(a, b), q), Pair.new(a.join(q.first), b.join(q.second)))
}

/// The componentwise join of an arbitrary pair with a `Pair.new` pair on the right.
theorem pair_join_new_right[A: JoinSemilattice, B: JoinSemilattice](p: Pair[A, B], a: A, b: B) {
    pair_join(p, Pair.new(a, b)) = Pair.new(p.first.join(a), p.second.join(b))
} by {
    pair_join_first(p, Pair.new(a, b))
    pair_join_second(p, Pair.new(a, b))
    pair_new_first(a, b)
    pair_new_second(a, b)
    pair_join(p, Pair.new(a, b)).first = p.first.join(a)
    pair_join(p, Pair.new(a, b)).second = p.second.join(b)
    pair_new_first(p.first.join(a), p.second.join(b))
    pair_new_second(p.first.join(a), p.second.join(b))
    Pair.new(p.first.join(a), p.second.join(b)).first = p.first.join(a)
    Pair.new(p.first.join(a), p.second.join(b)).second = p.second.join(b)
    pair_ext(pair_join(p, Pair.new(a, b)), Pair.new(p.first.join(a), p.second.join(b)))
}

/// First component of the componentwise meet with a `Pair.new` pair on the left.
theorem pair_meet_new_left_first[A: MeetSemilattice, B: MeetSemilattice](a: A, b: B, q: Pair[A, B]) {
    pair_meet(Pair.new(a, b), q).first = a.meet(q.first)
} by {
    pair_meet_first(Pair.new(a, b), q)
    pair_new_first(a, b)
}

/// Second component of the componentwise meet with a `Pair.new` pair on the left.
theorem pair_meet_new_left_second[A: MeetSemilattice, B: MeetSemilattice](a: A, b: B, q: Pair[A, B]) {
    pair_meet(Pair.new(a, b), q).second = b.meet(q.second)
} by {
    pair_meet_second(Pair.new(a, b), q)
    pair_new_second(a, b)
}

/// First component of the componentwise meet with a `Pair.new` pair on the right.
theorem pair_meet_new_right_first[A: MeetSemilattice, B: MeetSemilattice](p: Pair[A, B], a: A, b: B) {
    pair_meet(p, Pair.new(a, b)).first = p.first.meet(a)
} by {
    pair_meet_first(p, Pair.new(a, b))
    pair_new_first(a, b)
}

/// Second component of the componentwise meet with a `Pair.new` pair on the right.
theorem pair_meet_new_right_second[A: MeetSemilattice, B: MeetSemilattice](p: Pair[A, B], a: A, b: B) {
    pair_meet(p, Pair.new(a, b)).second = p.second.meet(b)
} by {
    pair_meet_second(p, Pair.new(a, b))
    pair_new_second(a, b)
}

/// First component of the componentwise join with a `Pair.new` pair on the left.
theorem pair_join_new_left_first[A: JoinSemilattice, B: JoinSemilattice](a: A, b: B, q: Pair[A, B]) {
    pair_join(Pair.new(a, b), q).first = a.join(q.first)
} by {
    pair_join_first(Pair.new(a, b), q)
    pair_new_first(a, b)
}

/// Second component of the componentwise join with a `Pair.new` pair on the left.
theorem pair_join_new_left_second[A: JoinSemilattice, B: JoinSemilattice](a: A, b: B, q: Pair[A, B]) {
    pair_join(Pair.new(a, b), q).second = b.join(q.second)
} by {
    pair_join_second(Pair.new(a, b), q)
    pair_new_second(a, b)
}

/// First component of the componentwise join with a `Pair.new` pair on the right.
theorem pair_join_new_right_first[A: JoinSemilattice, B: JoinSemilattice](p: Pair[A, B], a: A, b: B) {
    pair_join(p, Pair.new(a, b)).first = p.first.join(a)
} by {
    pair_join_first(p, Pair.new(a, b))
    pair_new_first(a, b)
}

/// Second component of the componentwise join with a `Pair.new` pair on the right.
theorem pair_join_new_right_second[A: JoinSemilattice, B: JoinSemilattice](p: Pair[A, B], a: A, b: B) {
    pair_join(p, Pair.new(a, b)).second = p.second.join(b)
} by {
    pair_join_second(p, Pair.new(a, b))
    pair_new_second(a, b)
}

/// First component of `pair_meet_left_comm` written componentwise.
theorem pair_meet_left_comm_first[A: MeetSemilattice, B: MeetSemilattice](p: Pair[A, B], q: Pair[A, B], r: Pair[A, B]) {
    pair_meet(p, pair_meet(q, r)).first = q.first.meet(p.first.meet(r.first))
} by {
    pair_meet_left_comm(p, q, r)
    pair_meet_assoc_rev_first(q, p, r)
}

/// Second component of `pair_meet_left_comm` written componentwise.
theorem pair_meet_left_comm_second[A: MeetSemilattice, B: MeetSemilattice](p: Pair[A, B], q: Pair[A, B], r: Pair[A, B]) {
    pair_meet(p, pair_meet(q, r)).second = q.second.meet(p.second.meet(r.second))
} by {
    pair_meet_left_comm(p, q, r)
    pair_meet_assoc_rev_second(q, p, r)
}

/// First component of `pair_join_left_comm` written componentwise.
theorem pair_join_left_comm_first[A: JoinSemilattice, B: JoinSemilattice](p: Pair[A, B], q: Pair[A, B], r: Pair[A, B]) {
    pair_join(p, pair_join(q, r)).first = q.first.join(p.first.join(r.first))
} by {
    pair_join_left_comm(p, q, r)
    pair_join_assoc_rev_first(q, p, r)
}

/// Second component of `pair_join_left_comm` written componentwise.
theorem pair_join_left_comm_second[A: JoinSemilattice, B: JoinSemilattice](p: Pair[A, B], q: Pair[A, B], r: Pair[A, B]) {
    pair_join(p, pair_join(q, r)).second = q.second.join(p.second.join(r.second))
} by {
    pair_join_left_comm(p, q, r)
    pair_join_assoc_rev_second(q, p, r)
}

/// First component of `pair_meet_right_comm` written componentwise.
theorem pair_meet_right_comm_first[A: MeetSemilattice, B: MeetSemilattice](p: Pair[A, B], q: Pair[A, B], r: Pair[A, B]) {
    pair_meet(pair_meet(p, q), r).first = p.first.meet(r.first).meet(q.first)
} by {
    pair_meet_right_comm(p, q, r)
    pair_meet_assoc_first(p, r, q)
}

/// Second component of `pair_meet_right_comm` written componentwise.
theorem pair_meet_right_comm_second[A: MeetSemilattice, B: MeetSemilattice](p: Pair[A, B], q: Pair[A, B], r: Pair[A, B]) {
    pair_meet(pair_meet(p, q), r).second = p.second.meet(r.second).meet(q.second)
} by {
    pair_meet_right_comm(p, q, r)
    pair_meet_assoc_second(p, r, q)
}

/// First component of `pair_join_right_comm` written componentwise.
theorem pair_join_right_comm_first[A: JoinSemilattice, B: JoinSemilattice](p: Pair[A, B], q: Pair[A, B], r: Pair[A, B]) {
    pair_join(pair_join(p, q), r).first = p.first.join(r.first).join(q.first)
} by {
    pair_join_right_comm(p, q, r)
    pair_join_assoc_first(p, r, q)
}

/// Second component of `pair_join_right_comm` written componentwise.
theorem pair_join_right_comm_second[A: JoinSemilattice, B: JoinSemilattice](p: Pair[A, B], q: Pair[A, B], r: Pair[A, B]) {
    pair_join(pair_join(p, q), r).second = p.second.join(r.second).join(q.second)
} by {
    pair_join_right_comm(p, q, r)
    pair_join_assoc_second(p, r, q)
}

/// The componentwise order makes a pair of partially ordered types into an `LTE` type.
instance Pair[A: PartialOrder, B: PartialOrder]: LTE {
    define lte(self, other: Pair[A, B]) -> Bool {
        pair_lte(self, other)
    }
}

/// The componentwise order on `Pair.lte` agrees with `pair_lte`.
theorem pair_lte_eq_lte[A: PartialOrder, B: PartialOrder](p: Pair[A, B], q: Pair[A, B]) {
    (p <= q) = pair_lte(p, q)
}

/// A pair of partially ordered types is a partial order under the componentwise order.
instance Pair[A: PartialOrder, B: PartialOrder]: PartialOrder

/// A pair of meet semilattices has a componentwise binary meet.
instance Pair[A: MeetSemilattice, B: MeetSemilattice]: Meet {
    define meet(self, other: Pair[A, B]) -> Pair[A, B] {
        pair_meet(self, other)
    }
}

/// A pair of join semilattices has a componentwise binary join.
instance Pair[A: JoinSemilattice, B: JoinSemilattice]: Join {
    define join(self, other: Pair[A, B]) -> Pair[A, B] {
        pair_join(self, other)
    }
}

/// The componentwise meet on `Pair.meet` agrees with `pair_meet`.
theorem pair_meet_eq_meet[A: MeetSemilattice, B: MeetSemilattice](p: Pair[A, B], q: Pair[A, B]) {
    p.meet(q) = pair_meet(p, q)
}

theorem pair_instance_meet_lte_left[A: MeetSemilattice, B: MeetSemilattice](p: Pair[A, B], q: Pair[A, B]) {
    p.meet(q) <= p
} by {
    pair_meet_eq_meet(p, q)
    pair_meet_lte_left(p, q)
    pair_lte_eq_lte(pair_meet(p, q), p)
}

theorem pair_instance_meet_lte_right[A: MeetSemilattice, B: MeetSemilattice](p: Pair[A, B], q: Pair[A, B]) {
    p.meet(q) <= q
} by {
    pair_meet_eq_meet(p, q)
    pair_meet_lte_right(p, q)
    pair_lte_eq_lte(pair_meet(p, q), q)
}

theorem pair_instance_lte_meet_of_bounds[A: MeetSemilattice, B: MeetSemilattice](
    c: Pair[A, B], p: Pair[A, B], q: Pair[A, B]
) {
    c <= p and c <= q implies c <= p.meet(q)
} by {
    if c <= p and c <= q {
        pair_lte_eq_lte(c, p)
        pair_lte_eq_lte(c, q)
        pair_lte(c, p)
        pair_lte(c, q)
        pair_lte_meet_of_bounds(c, p, q)
        pair_lte(c, pair_meet(p, q))
        pair_lte_eq_lte(c, pair_meet(p, q))
        c <= pair_meet(p, q)
        pair_meet_eq_meet(p, q)
        c <= p.meet(q)
    }
}

/// A pair of meet semilattices is a meet semilattice under the componentwise meet.
instance Pair[A: MeetSemilattice, B: MeetSemilattice]: MeetSemilattice

/// The componentwise join on `Pair.join` agrees with `pair_join`.
theorem pair_join_eq_join[A: JoinSemilattice, B: JoinSemilattice](p: Pair[A, B], q: Pair[A, B]) {
    p.join(q) = pair_join(p, q)
}

theorem pair_instance_lte_join_left[A: JoinSemilattice, B: JoinSemilattice](p: Pair[A, B], q: Pair[A, B]) {
    p <= p.join(q)
} by {
    pair_join_eq_join(p, q)
    pair_lte_join_left(p, q)
    pair_lte_eq_lte(p, pair_join(p, q))
}

theorem pair_instance_lte_join_right[A: JoinSemilattice, B: JoinSemilattice](p: Pair[A, B], q: Pair[A, B]) {
    q <= p.join(q)
} by {
    pair_join_eq_join(p, q)
    pair_lte_join_right(p, q)
    pair_lte_eq_lte(q, pair_join(p, q))
}

theorem pair_instance_join_lte_of_bounds[A: JoinSemilattice, B: JoinSemilattice](
    p: Pair[A, B], q: Pair[A, B], c: Pair[A, B]
) {
    p <= c and q <= c implies p.join(q) <= c
} by {
    if p <= c and q <= c {
        pair_lte_eq_lte(p, c)
        pair_lte_eq_lte(q, c)
        pair_lte(p, c)
        pair_lte(q, c)
        pair_join_lte_of_bounds(p, q, c)
        pair_lte(pair_join(p, q), c)
        pair_lte_eq_lte(pair_join(p, q), c)
        pair_join(p, q) <= c
        pair_join_eq_join(p, q)
        p.join(q) <= c
    }
}

/// A pair of join semilattices is a join semilattice under the componentwise join.
instance Pair[A: JoinSemilattice, B: JoinSemilattice]: JoinSemilattice

/// A pair of lattices is a lattice under the componentwise meet and join.
instance Pair[A: Lattice, B: Lattice]: Lattice

theorem pair_instance_meet_join_distrib_left[A: DistribLattice, B: DistribLattice](
    p: Pair[A, B], q: Pair[A, B], r: Pair[A, B]
) {
    p.meet(q.join(r)) = p.meet(q).join(p.meet(r))
} by {
    pair_join_eq_join(q, r)
    pair_meet_eq_meet(p, pair_join(q, r))
    pair_meet_join_distrib_left(p, q, r)
    pair_meet_eq_meet(p, q)
    pair_meet_eq_meet(p, r)
    pair_join_eq_join(pair_meet(p, q), pair_meet(p, r))
}

theorem pair_instance_join_meet_distrib_left[A: DistribLattice, B: DistribLattice](
    p: Pair[A, B], q: Pair[A, B], r: Pair[A, B]
) {
    p.join(q.meet(r)) = p.join(q).meet(p.join(r))
} by {
    pair_meet_eq_meet(q, r)
    pair_join_eq_join(p, pair_meet(q, r))
    pair_join_meet_distrib_left(p, q, r)
    pair_join_eq_join(p, q)
    pair_join_eq_join(p, r)
    pair_meet_eq_meet(pair_join(p, q), pair_join(p, r))
}

/// A pair of distributive lattices is a distributive lattice under the componentwise meet and join.
instance Pair[A: DistribLattice, B: DistribLattice]: DistribLattice

/// A componentwise pair map of two order embeddings is an order embedding for the componentwise order.
theorem pair_map_is_order_embedding[A: PartialOrder, B: PartialOrder, C: PartialOrder, D: PartialOrder](
    f: A -> C,
    g: B -> D
) {
    is_order_embedding(f) and is_order_embedding(g) implies is_order_embedding(pair_map(f, g))
} by {
    if is_order_embedding(f) and is_order_embedding(g) {
        forall(x: Pair[A, B], y: Pair[A, B]) {
            pair_map_lte_iff_of_embeddings(f, g, x, y)
            pair_lte(pair_map(f, g, x), pair_map(f, g, y)) = pair_lte(x, y)
            pair_lte_eq_lte(pair_map(f, g, x), pair_map(f, g, y))
            pair_lte_eq_lte(x, y)
            pair_map(f, g)(x) = pair_map(f, g, x)
            pair_map(f, g)(y) = pair_map(f, g, y)
            (pair_map(f, g)(x) <= pair_map(f, g)(y)) = (x <= y)
        }
        is_order_embedding(pair_map(f, g))
    }
}

/// Componentwise pair maps of two order-isomorphism pairs form an order-isomorphism pair.
theorem pair_map_is_order_iso_pair[A: PartialOrder, B: PartialOrder, C: PartialOrder, D: PartialOrder](
    f1: A -> C,
    g1: C -> A,
    f2: B -> D,
    g2: D -> B
) {
    is_order_iso_pair(f1, g1) and is_order_iso_pair(f2, g2) implies
    is_order_iso_pair(pair_map(f1, f2), pair_map(g1, g2))
} by {
    if is_order_iso_pair(f1, g1) and is_order_iso_pair(f2, g2) {
        forall(p: Pair[A, B]) {
            pair_map_comp(g1, f1, g2, f2, p)
            pair_map(compose(g1, f1), compose(g2, f2), p) = pair_map(g1, g2, pair_map(f1, f2, p))
            order_iso_pair_left_inverse(f1, g1, p.first)
            g1(f1(p.first)) = p.first
            order_iso_pair_left_inverse(f2, g2, p.second)
            g2(f2(p.second)) = p.second
            compose(g1, f1, p.first) = p.first
            compose(g2, f2, p.second) = p.second
            pair_map(compose(g1, f1), compose(g2, f2), p).first = p.first
            pair_map(compose(g1, f1), compose(g2, f2), p).second = p.second
            pair_ext(pair_map(compose(g1, f1), compose(g2, f2), p), p)
            pair_map(compose(g1, f1), compose(g2, f2), p) = p
            pair_map(g1, g2, pair_map(f1, f2, p)) = p
            pair_map(g1, g2)(pair_map(f1, f2)(p)) = p
        }
        forall(q: Pair[C, D]) {
            pair_map_comp(f1, g1, f2, g2, q)
            pair_map(compose(f1, g1), compose(f2, g2), q) = pair_map(f1, f2, pair_map(g1, g2, q))
            order_iso_pair_right_inverse(f1, g1, q.first)
            f1(g1(q.first)) = q.first
            order_iso_pair_right_inverse(f2, g2, q.second)
            f2(g2(q.second)) = q.second
            compose(f1, g1, q.first) = q.first
            compose(f2, g2, q.second) = q.second
            pair_map(compose(f1, g1), compose(f2, g2), q).first = q.first
            pair_map(compose(f1, g1), compose(f2, g2), q).second = q.second
            pair_ext(pair_map(compose(f1, g1), compose(f2, g2), q), q)
            pair_map(compose(f1, g1), compose(f2, g2), q) = q
            pair_map(f1, f2, pair_map(g1, g2, q)) = q
            pair_map(f1, f2)(pair_map(g1, g2)(q)) = q
        }
        order_iso_pair_map_is_order_embedding(f1, g1)
        is_order_embedding(f1)
        order_iso_pair_map_is_order_embedding(f2, g2)
        is_order_embedding(f2)
        pair_map_is_order_embedding(f1, f2)
        is_order_embedding(pair_map(f1, f2))
        order_iso_pair_inv_is_order_embedding(f1, g1)
        is_order_embedding(g1)
        order_iso_pair_inv_is_order_embedding(f2, g2)
        is_order_embedding(g2)
        pair_map_is_order_embedding(g1, g2)
        is_order_embedding(pair_map(g1, g2))
        is_order_iso_pair(pair_map(f1, f2), pair_map(g1, g2)) = (forall(a: Pair[A, B]) {
            pair_map(g1, g2)(pair_map(f1, f2)(a)) = a
        } and forall(b: Pair[C, D]) {
            pair_map(f1, f2)(pair_map(g1, g2)(b)) = b
        } and is_order_embedding(pair_map(f1, f2)) and is_order_embedding(pair_map(g1, g2)))
        forall(a: Pair[A, B]) {
            pair_map(g1, g2)(pair_map(f1, f2)(a)) = a
        }
        forall(b: Pair[C, D]) {
            pair_map(f1, f2)(pair_map(g1, g2)(b)) = b
        }
        is_order_embedding(pair_map(f1, f2)) and is_order_embedding(pair_map(g1, g2))
        (forall(b: Pair[C, D]) {
            pair_map(f1, f2)(pair_map(g1, g2)(b)) = b
        } and is_order_embedding(pair_map(f1, f2)) and is_order_embedding(pair_map(g1, g2)))
        (forall(a: Pair[A, B]) {
            pair_map(g1, g2)(pair_map(f1, f2)(a)) = a
        } and forall(b: Pair[C, D]) {
            pair_map(f1, f2)(pair_map(g1, g2)(b)) = b
        } and is_order_embedding(pair_map(f1, f2)) and is_order_embedding(pair_map(g1, g2)))
        is_order_iso_pair(pair_map(f1, f2), pair_map(g1, g2))
    }
}

/// The componentwise product of two order isomorphisms, as a bundled order isomorphism.
let pair_order_iso[A: PartialOrder, B: PartialOrder, C: PartialOrder, D: PartialOrder](e: OrderIso[A, C], h: OrderIso[B, D]) -> result: OrderIso[Pair[A, B], Pair[C, D]] satisfy {
    result.map = pair_map(e.map, h.map) and result.inv = pair_map(e.inv, h.inv)
} by {
    OrderIso[A, C].constraint(e.map, e.inv)
    is_order_iso_pair(e.map, e.inv)
    OrderIso[B, D].constraint(h.map, h.inv)
    is_order_iso_pair(h.map, h.inv)
    pair_map_is_order_iso_pair(e.map, e.inv, h.map, h.inv)
    is_order_iso_pair(pair_map(e.map, h.map), pair_map(e.inv, h.inv))
    let r: OrderIso[Pair[A, B], Pair[C, D]] satisfy {
        OrderIso[Pair[A, B], Pair[C, D]].new(pair_map(e.map, h.map), pair_map(e.inv, h.inv)) = Option.some(r)
    }
    order_iso_new_map(pair_map(e.map, h.map), pair_map(e.inv, h.inv), r)
    r.map = pair_map(e.map, h.map)
    order_iso_new_inv(pair_map(e.map, h.map), pair_map(e.inv, h.inv), r)
    r.inv = pair_map(e.inv, h.inv)
}

/// The map of the product order isomorphism is the componentwise pair map of the two maps.
theorem pair_order_iso_map[A: PartialOrder, B: PartialOrder, C: PartialOrder, D: PartialOrder](
    e: OrderIso[A, C],
    h: OrderIso[B, D]
) {
    pair_order_iso(e, h).map = pair_map(e.map, h.map)
}

/// The inverse of the product order isomorphism is the componentwise pair map of the two inverses.
theorem pair_order_iso_inv[A: PartialOrder, B: PartialOrder, C: PartialOrder, D: PartialOrder](
    e: OrderIso[A, C],
    h: OrderIso[B, D]
) {
    pair_order_iso(e, h).inv = pair_map(e.inv, h.inv)
}

/// The componentwise pair map of two identity maps is the identity map on pairs.
theorem pair_map_identity_fn[A, B] {
    pair_map(identity_fn[A], identity_fn[B]) = identity_fn[Pair[A, B]]
} by {
    forall(p: Pair[A, B]) {
        pair_map_identity(p)
        pair_map(identity_fn[A], identity_fn[B], p) = p
        identity_fn[Pair[A, B]](p) = p
        pair_map(identity_fn[A], identity_fn[B])(p) = identity_fn[Pair[A, B]](p)
    }
    function_extensionality(pair_map(identity_fn[A], identity_fn[B]), identity_fn[Pair[A, B]])
}

/// The componentwise pair map of two composites is the composite of the two pair maps.
theorem pair_map_compose_fn[A, B, C, D, E, F](f: C -> E, g: A -> C, h: D -> F, i: B -> D) {
    pair_map(compose(f, g), compose(h, i)) = compose(pair_map(f, h), pair_map(g, i))
} by {
    forall(p: Pair[A, B]) {
        pair_map_comp(f, g, h, i, p)
        pair_map(compose(f, g), compose(h, i), p) = pair_map(f, h, pair_map(g, i, p))
        compose(pair_map(f, h), pair_map(g, i), p) = pair_map(f, h)(pair_map(g, i)(p))
        pair_map(f, h)(pair_map(g, i)(p)) = pair_map(f, h, pair_map(g, i, p))
        pair_map(compose(f, g), compose(h, i))(p) = compose(pair_map(f, h), pair_map(g, i))(p)
    }
    function_extensionality(pair_map(compose(f, g), compose(h, i)), compose(pair_map(f, h), pair_map(g, i)))
}

/// The product of two identity order isomorphisms is the identity order isomorphism on pairs.
theorem pair_order_iso_identity[A: PartialOrder, B: PartialOrder](a: A, b: B) {
    pair_order_iso(identity_order_iso(a), identity_order_iso(b)) = identity_order_iso(Pair.new(a, b))
} by {
    pair_order_iso_map(identity_order_iso(a), identity_order_iso(b))
    pair_order_iso(identity_order_iso(a), identity_order_iso(b)).map =
        pair_map(identity_order_iso(a).map, identity_order_iso(b).map)
    identity_order_iso_map(a)
    identity_order_iso_map(b)
    pair_order_iso(identity_order_iso(a), identity_order_iso(b)).map =
        pair_map(identity_fn[A], identity_fn[B])
    pair_map_identity_fn[A, B]
    pair_order_iso(identity_order_iso(a), identity_order_iso(b)).map = identity_fn[Pair[A, B]]
    identity_order_iso_map(Pair.new(a, b))
    identity_order_iso(Pair.new(a, b)).map = identity_fn[Pair[A, B]]
    pair_order_iso(identity_order_iso(a), identity_order_iso(b)).map =
        identity_order_iso(Pair.new(a, b)).map
    pair_order_iso_inv(identity_order_iso(a), identity_order_iso(b))
    pair_order_iso(identity_order_iso(a), identity_order_iso(b)).inv =
        pair_map(identity_order_iso(a).inv, identity_order_iso(b).inv)
    identity_order_iso_inv(a)
    identity_order_iso_inv(b)
    pair_order_iso(identity_order_iso(a), identity_order_iso(b)).inv =
        pair_map(identity_fn[A], identity_fn[B])
    pair_order_iso(identity_order_iso(a), identity_order_iso(b)).inv = identity_fn[Pair[A, B]]
    identity_order_iso_inv(Pair.new(a, b))
    identity_order_iso(Pair.new(a, b)).inv = identity_fn[Pair[A, B]]
    pair_order_iso(identity_order_iso(a), identity_order_iso(b)).inv =
        identity_order_iso(Pair.new(a, b)).inv
    order_iso_ext(pair_order_iso(identity_order_iso(a), identity_order_iso(b)),
        identity_order_iso(Pair.new(a, b)))
}

/// The inverse of a product order isomorphism is the product of the two inverse order isomorphisms.
theorem pair_order_iso_inverse[A: PartialOrder, B: PartialOrder, C: PartialOrder, D: PartialOrder](
    e: OrderIso[A, C],
    h: OrderIso[B, D]
) {
    inverse_order_iso(pair_order_iso(e, h)) =
        pair_order_iso(inverse_order_iso(e), inverse_order_iso(h))
} by {
    inverse_order_iso_map(pair_order_iso(e, h))
    inverse_order_iso(pair_order_iso(e, h)).map = pair_order_iso(e, h).inv
    pair_order_iso_inv(e, h)
    inverse_order_iso(pair_order_iso(e, h)).map = pair_map(e.inv, h.inv)
    pair_order_iso_map(inverse_order_iso(e), inverse_order_iso(h))
    pair_order_iso(inverse_order_iso(e), inverse_order_iso(h)).map =
        pair_map(inverse_order_iso(e).map, inverse_order_iso(h).map)
    inverse_order_iso_map(e)
    inverse_order_iso_map(h)
    pair_order_iso(inverse_order_iso(e), inverse_order_iso(h)).map = pair_map(e.inv, h.inv)
    inverse_order_iso(pair_order_iso(e, h)).map =
        pair_order_iso(inverse_order_iso(e), inverse_order_iso(h)).map
    inverse_order_iso_inv(pair_order_iso(e, h))
    inverse_order_iso(pair_order_iso(e, h)).inv = pair_order_iso(e, h).map
    pair_order_iso_map(e, h)
    inverse_order_iso(pair_order_iso(e, h)).inv = pair_map(e.map, h.map)
    pair_order_iso_inv(inverse_order_iso(e), inverse_order_iso(h))
    pair_order_iso(inverse_order_iso(e), inverse_order_iso(h)).inv =
        pair_map(inverse_order_iso(e).inv, inverse_order_iso(h).inv)
    inverse_order_iso_inv(e)
    inverse_order_iso_inv(h)
    pair_order_iso(inverse_order_iso(e), inverse_order_iso(h)).inv = pair_map(e.map, h.map)
    inverse_order_iso(pair_order_iso(e, h)).inv =
        pair_order_iso(inverse_order_iso(e), inverse_order_iso(h)).inv
    order_iso_ext(inverse_order_iso(pair_order_iso(e, h)),
        pair_order_iso(inverse_order_iso(e), inverse_order_iso(h)))
}

/// Composing two product order isomorphisms is the product of the two composite order isomorphisms.
theorem pair_order_iso_compose[A1: PartialOrder, A2: PartialOrder, B1: PartialOrder, B2: PartialOrder, C1: PartialOrder, C2: PartialOrder](
    e2: OrderIso[B1, C1],
    h2: OrderIso[B2, C2],
    e1: OrderIso[A1, B1],
    h1: OrderIso[A2, B2]
) {
    compose_order_iso(pair_order_iso(e2, h2), pair_order_iso(e1, h1)) =
        pair_order_iso(compose_order_iso(e2, e1), compose_order_iso(h2, h1))
} by {
    compose_order_iso_map(pair_order_iso(e2, h2), pair_order_iso(e1, h1))
    compose_order_iso(pair_order_iso(e2, h2), pair_order_iso(e1, h1)).map =
        compose(pair_order_iso(e2, h2).map, pair_order_iso(e1, h1).map)
    pair_order_iso_map(e2, h2)
    pair_order_iso_map(e1, h1)
    compose_order_iso(pair_order_iso(e2, h2), pair_order_iso(e1, h1)).map =
        compose(pair_map(e2.map, h2.map), pair_map(e1.map, h1.map))
    pair_map_compose_fn(e2.map, e1.map, h2.map, h1.map)
    pair_map(compose(e2.map, e1.map), compose(h2.map, h1.map)) =
        compose(pair_map(e2.map, h2.map), pair_map(e1.map, h1.map))
    pair_order_iso_map(compose_order_iso(e2, e1), compose_order_iso(h2, h1))
    pair_order_iso(compose_order_iso(e2, e1), compose_order_iso(h2, h1)).map =
        pair_map(compose_order_iso(e2, e1).map, compose_order_iso(h2, h1).map)
    compose_order_iso_map(e2, e1)
    compose_order_iso_map(h2, h1)
    pair_order_iso(compose_order_iso(e2, e1), compose_order_iso(h2, h1)).map =
        pair_map(compose(e2.map, e1.map), compose(h2.map, h1.map))
    compose_order_iso(pair_order_iso(e2, h2), pair_order_iso(e1, h1)).map =
        pair_order_iso(compose_order_iso(e2, e1), compose_order_iso(h2, h1)).map
    compose_order_iso_inv(pair_order_iso(e2, h2), pair_order_iso(e1, h1))
    compose_order_iso(pair_order_iso(e2, h2), pair_order_iso(e1, h1)).inv =
        compose(pair_order_iso(e1, h1).inv, pair_order_iso(e2, h2).inv)
    pair_order_iso_inv(e1, h1)
    pair_order_iso_inv(e2, h2)
    compose_order_iso(pair_order_iso(e2, h2), pair_order_iso(e1, h1)).inv =
        compose(pair_map(e1.inv, h1.inv), pair_map(e2.inv, h2.inv))
    pair_map_compose_fn(e1.inv, e2.inv, h1.inv, h2.inv)
    pair_map(compose(e1.inv, e2.inv), compose(h1.inv, h2.inv)) =
        compose(pair_map(e1.inv, h1.inv), pair_map(e2.inv, h2.inv))
    pair_order_iso_inv(compose_order_iso(e2, e1), compose_order_iso(h2, h1))
    pair_order_iso(compose_order_iso(e2, e1), compose_order_iso(h2, h1)).inv =
        pair_map(compose_order_iso(e2, e1).inv, compose_order_iso(h2, h1).inv)
    compose_order_iso_inv(e2, e1)
    compose_order_iso_inv(h2, h1)
    pair_order_iso(compose_order_iso(e2, e1), compose_order_iso(h2, h1)).inv =
        pair_map(compose(e1.inv, e2.inv), compose(h1.inv, h2.inv))
    compose_order_iso(pair_order_iso(e2, h2), pair_order_iso(e1, h1)).inv =
        pair_order_iso(compose_order_iso(e2, e1), compose_order_iso(h2, h1)).inv
    order_iso_ext(compose_order_iso(pair_order_iso(e2, h2), pair_order_iso(e1, h1)),
        pair_order_iso(compose_order_iso(e2, e1), compose_order_iso(h2, h1)))
}

/// Applying a product order isomorphism applies the two component isomorphisms componentwise.
theorem pair_order_iso_apply[A: PartialOrder, B: PartialOrder, C: PartialOrder, D: PartialOrder](
    e: OrderIso[A, C],
    h: OrderIso[B, D],
    p: Pair[A, B]
) {
    order_iso_apply(pair_order_iso(e, h), p) =
        Pair.new(order_iso_apply(e, p.first), order_iso_apply(h, p.second))
} by {
    order_iso_apply_at(pair_order_iso(e, h), p)
    order_iso_apply(pair_order_iso(e, h), p) = pair_order_iso(e, h).map(p)
    pair_order_iso_map(e, h)
    pair_order_iso(e, h).map = pair_map(e.map, h.map)
    order_iso_apply(pair_order_iso(e, h), p) = pair_map(e.map, h.map, p)
    pair_map(e.map, h.map, p) = Pair.new(e.map(p.first), h.map(p.second))
    order_iso_apply_at(e, p.first)
    order_iso_apply(e, p.first) = e.map(p.first)
    order_iso_apply_at(h, p.second)
    order_iso_apply(h, p.second) = h.map(p.second)
    order_iso_apply(pair_order_iso(e, h), p) =
        Pair.new(order_iso_apply(e, p.first), order_iso_apply(h, p.second))
}

/// Unapplying a product order isomorphism unapplies the two component isomorphisms componentwise.
theorem pair_order_iso_unapply[A: PartialOrder, B: PartialOrder, C: PartialOrder, D: PartialOrder](
    e: OrderIso[A, C],
    h: OrderIso[B, D],
    q: Pair[C, D]
) {
    order_iso_unapply(pair_order_iso(e, h), q) =
        Pair.new(order_iso_unapply(e, q.first), order_iso_unapply(h, q.second))
} by {
    order_iso_unapply_at(pair_order_iso(e, h), q)
    order_iso_unapply(pair_order_iso(e, h), q) = pair_order_iso(e, h).inv(q)
    pair_order_iso_inv(e, h)
    pair_order_iso(e, h).inv = pair_map(e.inv, h.inv)
    order_iso_unapply(pair_order_iso(e, h), q) = pair_map(e.inv, h.inv, q)
    pair_map(e.inv, h.inv, q) = Pair.new(e.inv(q.first), h.inv(q.second))
    order_iso_unapply_at(e, q.first)
    order_iso_unapply(e, q.first) = e.inv(q.first)
    order_iso_unapply_at(h, q.second)
    order_iso_unapply(h, q.second) = h.inv(q.second)
    order_iso_unapply(pair_order_iso(e, h), q) =
        Pair.new(order_iso_unapply(e, q.first), order_iso_unapply(h, q.second))
}

/// A product order isomorphism applied to a constructed pair builds the constructed image pair.
theorem pair_order_iso_apply_new[A: PartialOrder, B: PartialOrder, C: PartialOrder, D: PartialOrder](
    e: OrderIso[A, C],
    h: OrderIso[B, D],
    a: A,
    b: B
) {
    order_iso_apply(pair_order_iso(e, h), Pair.new(a, b)) =
        Pair.new(order_iso_apply(e, a), order_iso_apply(h, b))
} by {
    pair_order_iso_apply(e, h, Pair.new(a, b))
    pair_new_first(a, b)
    Pair.new(a, b).first = a
    pair_new_second(a, b)
    Pair.new(a, b).second = b
}

/// Swapping the components is an order embedding for the componentwise order.
theorem pair_swap_map_is_order_embedding[A: PartialOrder, B: PartialOrder] {
    is_order_embedding(Pair[A, B].swap)
} by {
    forall(x: Pair[A, B], y: Pair[A, B]) {
        pair_swap_lte_iff(x, y)
        pair_lte(x.swap, y.swap) = pair_lte(x, y)
        pair_lte_eq_lte(x.swap, y.swap)
        pair_lte_eq_lte(x, y)
        (x.swap <= y.swap) = (x <= y)
        Pair[A, B].swap(x) = x.swap
        Pair[A, B].swap(y) = y.swap
        (Pair[A, B].swap(x) <= Pair[A, B].swap(y)) = (x <= y)
    }
}

/// Swapping in each direction forms an order isomorphism pair.
theorem pair_swap_is_order_iso_pair[A: PartialOrder, B: PartialOrder] {
    is_order_iso_pair(Pair[A, B].swap, Pair[B, A].swap)
} by {
    forall(p: Pair[A, B]) {
        swap_swap(p)
        Pair[B, A].swap(Pair[A, B].swap(p)) = p.swap.swap
        Pair[B, A].swap(Pair[A, B].swap(p)) = p
    }
    forall(q: Pair[B, A]) {
        swap_swap(q)
        Pair[A, B].swap(Pair[B, A].swap(q)) = q.swap.swap
        Pair[A, B].swap(Pair[B, A].swap(q)) = q
    }
    pair_swap_map_is_order_embedding[A, B]
    is_order_embedding(Pair[A, B].swap)
    pair_swap_map_is_order_embedding[B, A]
    is_order_embedding(Pair[B, A].swap)
    is_order_iso_pair(Pair[A, B].swap, Pair[B, A].swap) = (forall(a: Pair[A, B]) {
        Pair[B, A].swap(Pair[A, B].swap(a)) = a
    } and forall(b: Pair[B, A]) {
        Pair[A, B].swap(Pair[B, A].swap(b)) = b
    } and is_order_embedding(Pair[A, B].swap) and is_order_embedding(Pair[B, A].swap))
    forall(a: Pair[A, B]) {
        Pair[B, A].swap(Pair[A, B].swap(a)) = a
    }
    forall(b: Pair[B, A]) {
        Pair[A, B].swap(Pair[B, A].swap(b)) = b
    }
    is_order_embedding(Pair[A, B].swap) and is_order_embedding(Pair[B, A].swap)
    (forall(b: Pair[B, A]) {
        Pair[A, B].swap(Pair[B, A].swap(b)) = b
    } and is_order_embedding(Pair[A, B].swap) and is_order_embedding(Pair[B, A].swap))
    (forall(a: Pair[A, B]) {
        Pair[B, A].swap(Pair[A, B].swap(a)) = a
    } and forall(b: Pair[B, A]) {
        Pair[A, B].swap(Pair[B, A].swap(b)) = b
    } and is_order_embedding(Pair[A, B].swap) and is_order_embedding(Pair[B, A].swap))
    is_order_iso_pair(Pair[A, B].swap, Pair[B, A].swap)
}

/// Swapping the components of a pair, as a bundled order isomorphism.
let pair_swap_order_iso[A: PartialOrder, B: PartialOrder](anchor: Pair[A, B]) -> result: OrderIso[Pair[A, B], Pair[B, A]] satisfy {
    result.map = Pair[A, B].swap and result.inv = Pair[B, A].swap
} by {
    pair_swap_is_order_iso_pair[A, B]
    is_order_iso_pair(Pair[A, B].swap, Pair[B, A].swap)
    let r: OrderIso[Pair[A, B], Pair[B, A]] satisfy {
        OrderIso[Pair[A, B], Pair[B, A]].new(Pair[A, B].swap, Pair[B, A].swap) = Option.some(r)
    }
    order_iso_new_map(Pair[A, B].swap, Pair[B, A].swap, r)
    r.map = Pair[A, B].swap
    order_iso_new_inv(Pair[A, B].swap, Pair[B, A].swap, r)
    r.inv = Pair[B, A].swap
}

/// The map of the swap order isomorphism is the swap function.
theorem pair_swap_order_iso_map[A: PartialOrder, B: PartialOrder](anchor: Pair[A, B]) {
    pair_swap_order_iso(anchor).map = Pair[A, B].swap
}

/// The inverse of the swap order isomorphism is the swap function in the other direction.
theorem pair_swap_order_iso_inv[A: PartialOrder, B: PartialOrder](anchor: Pair[A, B]) {
    pair_swap_order_iso(anchor).inv = Pair[B, A].swap
}

/// Applying the swap order isomorphism swaps the components.
theorem pair_swap_order_iso_apply[A: PartialOrder, B: PartialOrder](p: Pair[A, B]) {
    order_iso_apply(pair_swap_order_iso(p), p) = p.swap
} by {
    order_iso_apply_at(pair_swap_order_iso(p), p)
    order_iso_apply(pair_swap_order_iso(p), p) = pair_swap_order_iso(p).map(p)
    pair_swap_order_iso_map(p)
    pair_swap_order_iso(p).map = Pair[A, B].swap
    order_iso_apply(pair_swap_order_iso(p), p) = Pair[A, B].swap(p)
    Pair[A, B].swap(p) = p.swap
}

/// Unapplying the swap order isomorphism swaps the components back.
theorem pair_swap_order_iso_unapply[A: PartialOrder, B: PartialOrder](anchor: Pair[A, B], q: Pair[B, A]) {
    order_iso_unapply(pair_swap_order_iso(anchor), q) = q.swap
} by {
    order_iso_unapply_at(pair_swap_order_iso(anchor), q)
    order_iso_unapply(pair_swap_order_iso(anchor), q) = pair_swap_order_iso(anchor).inv(q)
    pair_swap_order_iso_inv(anchor)
    pair_swap_order_iso(anchor).inv = Pair[B, A].swap
    order_iso_unapply(pair_swap_order_iso(anchor), q) = Pair[B, A].swap(q)
    Pair[B, A].swap(q) = q.swap
}

/// The inverse of the swap order isomorphism is the swap order isomorphism in the other direction.
theorem pair_swap_order_iso_inverse[A: PartialOrder, B: PartialOrder](anchor: Pair[A, B]) {
    inverse_order_iso(pair_swap_order_iso(anchor)) = pair_swap_order_iso(anchor.swap)
} by {
    inverse_order_iso_map(pair_swap_order_iso(anchor))
    inverse_order_iso(pair_swap_order_iso(anchor)).map = pair_swap_order_iso(anchor).inv
    pair_swap_order_iso_inv(anchor)
    inverse_order_iso(pair_swap_order_iso(anchor)).map = Pair[B, A].swap
    pair_swap_order_iso_map(anchor.swap)
    pair_swap_order_iso(anchor.swap).map = Pair[B, A].swap
    inverse_order_iso(pair_swap_order_iso(anchor)).map = pair_swap_order_iso(anchor.swap).map
    inverse_order_iso_inv(pair_swap_order_iso(anchor))
    inverse_order_iso(pair_swap_order_iso(anchor)).inv = pair_swap_order_iso(anchor).map
    pair_swap_order_iso_map(anchor)
    inverse_order_iso(pair_swap_order_iso(anchor)).inv = Pair[A, B].swap
    pair_swap_order_iso_inv(anchor.swap)
    pair_swap_order_iso(anchor.swap).inv = Pair[A, B].swap
    inverse_order_iso(pair_swap_order_iso(anchor)).inv = pair_swap_order_iso(anchor.swap).inv
    order_iso_ext(inverse_order_iso(pair_swap_order_iso(anchor)), pair_swap_order_iso(anchor.swap))
}

/// The swap order isomorphism applied to a constructed pair builds the swapped constructed pair.
theorem pair_swap_order_iso_apply_new[A: PartialOrder, B: PartialOrder](a: A, b: B) {
    order_iso_apply(pair_swap_order_iso(Pair.new(a, b)), Pair.new(a, b)) = Pair.new(b, a)
} by {
    pair_swap_order_iso_apply(Pair.new(a, b))
    order_iso_apply(pair_swap_order_iso(Pair.new(a, b)), Pair.new(a, b)) = Pair.new(a, b).swap
    swap_first(Pair.new(a, b))
    swap_second(Pair.new(a, b))
    pair_new_first(a, b)
    pair_new_second(a, b)
    Pair.new(a, b).swap.first = b
    Pair.new(a, b).swap.second = a
    pair_new_first(b, a)
    pair_new_second(b, a)
    Pair.new(b, a).first = b
    Pair.new(b, a).second = a
    pair_ext(Pair.new(a, b).swap, Pair.new(b, a))
    Pair.new(a, b).swap = Pair.new(b, a)
}

/// Composing the swap order isomorphism with itself is the identity order isomorphism.
theorem pair_swap_order_iso_compose_self[A: PartialOrder, B: PartialOrder](anchor: Pair[A, B]) {
    compose_order_iso(pair_swap_order_iso(anchor.swap), pair_swap_order_iso(anchor)) =
        identity_order_iso(anchor)
} by {
    compose_order_iso_map(pair_swap_order_iso(anchor.swap), pair_swap_order_iso(anchor))
    compose_order_iso(pair_swap_order_iso(anchor.swap), pair_swap_order_iso(anchor)).map =
        compose(pair_swap_order_iso(anchor.swap).map, pair_swap_order_iso(anchor).map)
    pair_swap_order_iso_map(anchor.swap)
    pair_swap_order_iso_map(anchor)
    compose_order_iso(pair_swap_order_iso(anchor.swap), pair_swap_order_iso(anchor)).map =
        compose(Pair[B, A].swap, Pair[A, B].swap)
    forall(p: Pair[A, B]) {
        compose(Pair[B, A].swap, Pair[A, B].swap, p) = Pair[B, A].swap(Pair[A, B].swap(p))
        swap_swap(p)
        Pair[B, A].swap(Pair[A, B].swap(p)) = p.swap.swap
        Pair[B, A].swap(Pair[A, B].swap(p)) = p
        identity_fn[Pair[A, B]](p) = p
        compose(Pair[B, A].swap, Pair[A, B].swap)(p) = identity_fn[Pair[A, B]](p)
    }
    function_extensionality(compose(Pair[B, A].swap, Pair[A, B].swap), identity_fn[Pair[A, B]])
    compose(Pair[B, A].swap, Pair[A, B].swap) = identity_fn[Pair[A, B]]
    compose_order_iso(pair_swap_order_iso(anchor.swap), pair_swap_order_iso(anchor)).map =
        identity_fn[Pair[A, B]]
    identity_order_iso_map(anchor)
    identity_order_iso(anchor).map = identity_fn[Pair[A, B]]
    compose_order_iso(pair_swap_order_iso(anchor.swap), pair_swap_order_iso(anchor)).map =
        identity_order_iso(anchor).map
    compose_order_iso_inv(pair_swap_order_iso(anchor.swap), pair_swap_order_iso(anchor))
    compose_order_iso(pair_swap_order_iso(anchor.swap), pair_swap_order_iso(anchor)).inv =
        compose(pair_swap_order_iso(anchor).inv, pair_swap_order_iso(anchor.swap).inv)
    pair_swap_order_iso_inv(anchor)
    pair_swap_order_iso_inv(anchor.swap)
    compose_order_iso(pair_swap_order_iso(anchor.swap), pair_swap_order_iso(anchor)).inv =
        compose(Pair[B, A].swap, Pair[A, B].swap)
    compose_order_iso(pair_swap_order_iso(anchor.swap), pair_swap_order_iso(anchor)).inv =
        identity_fn[Pair[A, B]]
    identity_order_iso_inv(anchor)
    identity_order_iso(anchor).inv = identity_fn[Pair[A, B]]
    compose_order_iso(pair_swap_order_iso(anchor.swap), pair_swap_order_iso(anchor)).inv =
        identity_order_iso(anchor).inv
    order_iso_ext(compose_order_iso(pair_swap_order_iso(anchor.swap), pair_swap_order_iso(anchor)),
        identity_order_iso(anchor))
}
