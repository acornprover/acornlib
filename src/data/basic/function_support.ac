/// Support and multiplicative support of functions.
///
/// This file is a small Acorn-side translation slice of
/// `Mathlib/Algebra/Group/Support.lean` / `Mathlib.Algebra.Notation.Support`.
/// It provides the core set-valued support predicates and low-risk subset and
/// extensionality lemmas compatible with the existing pointwise function API.

from algebra.add_monoid import AddMonoid
from data.basic.function_algebra import pointwise_add, pointwise_mul
from algebra.monoid.monoid import Monoid
from algebra.one import One
from data.basic.set import Set, empty_set_contains_eq, not_union_contains_eq, set_ext, union_contains_eq
from algebra.zero import Zero

/// Predicate for the additive/support set of a function.
define function_support_contains[T, A: Zero](f: T -> A, t: T) -> Bool {
    f(t) != A.0
}

/// The additive/support set of a function: the points where `f` is nonzero.
define function_support[T, A: Zero](f: T -> A) -> Set[T] {
    Set[T].new(function_support_contains(f))
}

/// Predicate for the multiplicative support of a function.
define function_mul_support_contains[T, A: One](f: T -> A, t: T) -> Bool {
    f(t) != A.1
}

/// The multiplicative support of a function: the points where `f` is not one.
define function_mul_support[T, A: One](f: T -> A) -> Set[T] {
    Set[T].new(function_mul_support_contains(f))
}

/// Membership in the support is exactly nonzero value.
theorem function_support_contains_eq[T, A: Zero](f: T -> A, t: T) {
    function_support(f).contains(t) = (f(t) != A.0)
} by {
    function_support(f).contains(t) = function_support_contains(f, t)
    function_support_contains(f, t) = (f(t) != A.0)
}

/// Non-membership in the support is exactly zero value.
theorem function_support_not_contains_eq[T, A: Zero](f: T -> A, t: T) {
    not function_support(f).contains(t) = (f(t) = A.0)
} by {
    function_support_contains_eq(f, t)
}

/// Membership in the multiplicative support is exactly non-one value.
theorem function_mul_support_contains_eq[T, A: One](f: T -> A, t: T) {
    function_mul_support(f).contains(t) = (f(t) != A.1)
} by {
    function_mul_support(f).contains(t) = function_mul_support_contains(f, t)
    function_mul_support_contains(f, t) = (f(t) != A.1)
}

/// Non-membership in the multiplicative support is exactly one value.
theorem function_mul_support_not_contains_eq[T, A: One](f: T -> A, t: T) {
    not function_mul_support(f).contains(t) = (f(t) = A.1)
} by {
    function_mul_support_contains_eq(f, t)
}

/// If a function is zero outside `s`, its support is contained in `s`.
theorem function_support_subset_of_eq_zero_outside[T, A: Zero](f: T -> A, s: Set[T]) {
    (forall(t: T) { not s.contains(t) implies f(t) = A.0 }) implies
    function_support(f).subset(s)
} by {
    if forall(t: T) { not s.contains(t) implies f(t) = A.0 } {
        forall(t: T) {
            if function_support(f).contains(t) {
                function_support_contains_eq(f, t)
                f(t) != A.0
                if not s.contains(t) {
                    f(t) = A.0
                    false
                }
                s.contains(t)
            }
        }
    }
}

/// If a function is one outside `s`, its multiplicative support is contained in `s`.
theorem function_mul_support_subset_of_eq_one_outside[T, A: One](f: T -> A, s: Set[T]) {
    (forall(t: T) { not s.contains(t) implies f(t) = A.1 }) implies
    function_mul_support(f).subset(s)
} by {
    if forall(t: T) { not s.contains(t) implies f(t) = A.1 } {
        forall(t: T) {
            if function_mul_support(f).contains(t) {
                function_mul_support_contains_eq(f, t)
                f(t) != A.1
                if not s.contains(t) {
                    f(t) = A.1
                    false
                }
                s.contains(t)
            }
        }
    }
}

/// The support of a pointwise sum is contained in the union of the two supports.
theorem function_support_add_subset[T, A: AddMonoid](f: T -> A, g: T -> A) {
    function_support(pointwise_add(f, g)).subset(function_support(f).union(function_support(g)))
} by {
    forall(t: T) {
        if function_support(pointwise_add(f, g)).contains(t) {
            function_support_contains_eq(pointwise_add(f, g), t)
            pointwise_add(f, g, t) != A.0
            if not function_support(f).union(function_support(g)).contains(t) {
                not_union_contains_eq(function_support(f), function_support(g), t)
                not function_support(f).contains(t) and not function_support(g).contains(t)
                function_support_not_contains_eq(f, t)
                function_support_not_contains_eq(g, t)
                f(t) = A.0
                g(t) = A.0
                pointwise_add(f, g, t) = f(t) + g(t)
                pointwise_add(f, g, t) = A.0 + A.0
                A.0 + A.0 = A.0
                false
            }
            function_support(f).union(function_support(g)).contains(t)
        }
    }
}

/// The multiplicative support of a pointwise product is contained in the union of the two multiplicative supports.
theorem function_mul_support_mul_subset[T, A: Monoid](f: T -> A, g: T -> A) {
    function_mul_support(pointwise_mul(f, g)).subset(function_mul_support(f).union(function_mul_support(g)))
} by {
    forall(t: T) {
        if function_mul_support(pointwise_mul(f, g)).contains(t) {
            function_mul_support_contains_eq(pointwise_mul(f, g), t)
            pointwise_mul(f, g, t) != A.1
            if not function_mul_support(f).union(function_mul_support(g)).contains(t) {
                not_union_contains_eq(function_mul_support(f), function_mul_support(g), t)
                not function_mul_support(f).contains(t) and not function_mul_support(g).contains(t)
                function_mul_support_not_contains_eq(f, t)
                function_mul_support_not_contains_eq(g, t)
                f(t) = A.1
                g(t) = A.1
                pointwise_mul(f, g, t) = f(t) * g(t)
                pointwise_mul(f, g, t) = A.1 * A.1
                A.1 * A.1 = A.1
                false
            }
            function_mul_support(f).union(function_mul_support(g)).contains(t)
        }
    }
}
