from pair import Pair
from algebra.product_algebra import pair_add, pair_mul, pair_zero, pair_one, pair_neg,
    pair_add_assoc, pair_add_comm, pair_add_zero_right, pair_add_zero_left,
    pair_add_neg_right, pair_mul_assoc, pair_mul_comm, pair_mul_one_right,
    pair_mul_one_left, pair_mul_add_distrib_left, pair_mul_add_distrib_right,
    pair_mul_zero_right, pair_mul_zero_left
from algebra.add import Add
from algebra.zero import Zero
from algebra.neg import Neg
from algebra.mul import Mul
from algebra.one import One
from algebra.add_semigroup import AddSemigroup
from algebra.add_comm_semigroup import AddCommSemigroup
from algebra.add_monoid import AddMonoid
from algebra.add_comm_monoid import AddCommMonoid
from algebra.add_group import AddGroup
from algebra.add_comm_group import AddCommGroup
from algebra.semigroup import Semigroup
from algebra.comm_semigroup import CommSemigroup
from algebra.monoid.monoid import Monoid
from algebra.comm_monoid import CommMonoid
from semiring import Semiring
from algebra.ring.ring import Ring
from comm_ring import CommRing

/// Componentwise addition as the additive operation on a binary pair.
attributes Pair[A: Add, B: Add] {
    define add(self, other: Pair[A, B]) -> Pair[A, B] {
        pair_add(self, other)
    }
}

/// Componentwise multiplication as the multiplicative operation on a binary pair.
attributes Pair[A: Mul, B: Mul] {
    define mul(self, other: Pair[A, B]) -> Pair[A, B] {
        pair_mul(self, other)
    }
}

/// Componentwise zero as the additive identity on a binary pair.
attributes Pair[A: Zero, B: Zero] {
    let zero: Pair[A, B] = pair_zero[A, B]
}

/// Componentwise one as the multiplicative identity on a binary pair.
attributes Pair[A: One, B: One] {
    let one: Pair[A, B] = pair_one[A, B]
}

/// Componentwise negation as additive inverse on a binary pair.
attributes Pair[A: Neg, B: Neg] {
    define neg(self) -> Pair[A, B] {
        pair_neg(self)
    }
}

/// Binary pairs have componentwise addition.
instance Pair[A: Add, B: Add]: Add {
    let add = Pair[A, B].add
}

/// Binary pairs have componentwise zero.
instance Pair[A: Zero, B: Zero]: Zero {
    let 0 = Pair[A, B].zero
}

/// Binary pairs have componentwise multiplication.
instance Pair[A: Mul, B: Mul]: Mul {
    let mul = Pair[A, B].mul
}

/// Binary pairs have componentwise one.
instance Pair[A: One, B: One]: One {
    let 1 = Pair[A, B].one
}

/// Binary pairs have componentwise negation.
instance Pair[A: Neg, B: Neg]: Neg {
    let neg = Pair[A, B].neg
}

/// The typeclass addition on pairs is the componentwise product-algebra operation.
theorem pair_typeclass_add_eq_pair_add[A: Add, B: Add](p: Pair[A, B], q: Pair[A, B]) {
    p + q = pair_add(p, q)
}

/// The typeclass multiplication on pairs is the componentwise product-algebra operation.
theorem pair_typeclass_mul_eq_pair_mul[A: Mul, B: Mul](p: Pair[A, B], q: Pair[A, B]) {
    p * q = pair_mul(p, q)
}

/// The typeclass zero on pairs is the componentwise product-algebra zero.
theorem pair_typeclass_zero_eq_pair_zero[A: Zero, B: Zero] {
    Zero.0[Pair[A, B]] = pair_zero[A, B]
}

/// The typeclass one on pairs is the componentwise product-algebra one.
theorem pair_typeclass_one_eq_pair_one[A: One, B: One] {
    One.1[Pair[A, B]] = pair_one[A, B]
}

/// The typeclass negation on pairs is the componentwise product-algebra negation.
theorem pair_typeclass_neg_eq_pair_neg[A: Neg, B: Neg](p: Pair[A, B]) {
    -p = pair_neg(p)
}

/// Componentwise pair addition is associative for additive semigroup factors.
theorem pair_add_semigroup_law[A: AddSemigroup, B: AddSemigroup](
    p: Pair[A, B],
    q: Pair[A, B],
    r: Pair[A, B]
) {
    p + (q + r) = (p + q) + r
} by {
    pair_typeclass_add_eq_pair_add(q, r)
    q + r = pair_add(q, r)
    pair_typeclass_add_eq_pair_add(p, q + r)
    p + (q + r) = pair_add(p, q + r)
    p + (q + r) = pair_add(p, pair_add(q, r))

    pair_typeclass_add_eq_pair_add(p, q)
    p + q = pair_add(p, q)
    pair_typeclass_add_eq_pair_add(p + q, r)
    (p + q) + r = pair_add(p + q, r)
    (p + q) + r = pair_add(pair_add(p, q), r)

    pair_add_assoc(p, q, r)
}

/// Componentwise pair addition satisfies the exact additive-semigroup typeclass law.
theorem pair_add_semigroup_instance_law[A: AddSemigroup, B: AddSemigroup](
    p: Pair[A, B],
    q: Pair[A, B],
    r: Pair[A, B]
) {
    Add.add(p, Add.add(q, r)) = Add.add(Add.add(p, q), r)
} by {
    pair_add_semigroup_law(p, q, r)
}

/// Binary pairs of additive semigroups are additive semigroups.
instance Pair[A: AddSemigroup, B: AddSemigroup]: AddSemigroup

/// Componentwise pair addition is commutative for additive commutative semigroup factors.
theorem pair_add_comm_semigroup_law[A: AddCommSemigroup, B: AddCommSemigroup](
    p: Pair[A, B],
    q: Pair[A, B]
) {
    p + q = q + p
} by {
    pair_typeclass_add_eq_pair_add(p, q)
    p + q = pair_add(p, q)
    pair_typeclass_add_eq_pair_add(q, p)
    q + p = pair_add(q, p)
    pair_add_comm(p, q)
}

/// Componentwise pair addition satisfies the exact additive-commutative-semigroup law.
theorem pair_add_comm_semigroup_instance_law[A: AddCommSemigroup, B: AddCommSemigroup](
    p: Pair[A, B],
    q: Pair[A, B]
) {
    Add.add(p, q) = Add.add(q, p)
} by {
    pair_add_comm_semigroup_law(p, q)
}

/// Binary pairs of additive commutative semigroups are additive commutative semigroups.
instance Pair[A: AddCommSemigroup, B: AddCommSemigroup]: AddCommSemigroup

/// Componentwise zero is a right identity for componentwise pair addition.
theorem pair_add_monoid_right_law[A: AddMonoid, B: AddMonoid](p: Pair[A, B]) {
    p + Zero.0[Pair[A, B]] = p
} by {
    pair_typeclass_zero_eq_pair_zero[A, B]
    Zero.0[Pair[A, B]] = pair_zero[A, B]
    pair_typeclass_add_eq_pair_add(p, Zero.0[Pair[A, B]])
    p + Zero.0[Pair[A, B]] = pair_add(p, Zero.0[Pair[A, B]])
    p + Zero.0[Pair[A, B]] = pair_add(p, pair_zero[A, B])
    pair_add_zero_right(p)
}

/// Componentwise zero is a left identity for componentwise pair addition.
theorem pair_add_monoid_left_law[A: AddMonoid, B: AddMonoid](p: Pair[A, B]) {
    Zero.0[Pair[A, B]] + p = p
} by {
    pair_typeclass_zero_eq_pair_zero[A, B]
    Zero.0[Pair[A, B]] = pair_zero[A, B]
    pair_typeclass_add_eq_pair_add(Zero.0[Pair[A, B]], p)
    Zero.0[Pair[A, B]] + p = pair_add(Zero.0[Pair[A, B]], p)
    Zero.0[Pair[A, B]] + p = pair_add(pair_zero[A, B], p)
    pair_add_zero_left(p)
}

/// Binary pairs of additive monoids are additive monoids.
instance Pair[A: AddMonoid, B: AddMonoid]: AddMonoid

/// Binary pairs of additive commutative monoids are additive commutative monoids.
instance Pair[A: AddCommMonoid, B: AddCommMonoid]: AddCommMonoid

/// Componentwise negation is a right additive inverse for componentwise pair addition.
theorem pair_add_group_inverse_law[A: AddGroup, B: AddGroup](p: Pair[A, B]) {
    p + -p = Zero.0[Pair[A, B]]
} by {
    pair_typeclass_neg_eq_pair_neg(p)
    -p = pair_neg(p)
    pair_typeclass_add_eq_pair_add(p, -p)
    p + -p = pair_add(p, -p)
    p + -p = pair_add(p, pair_neg(p))
    pair_typeclass_zero_eq_pair_zero[A, B]
    Zero.0[Pair[A, B]] = pair_zero[A, B]
    pair_add_neg_right(p)
}

/// Componentwise pair negation satisfies the exact additive-group inverse law.
theorem pair_add_group_instance_law[A: AddGroup, B: AddGroup](p: Pair[A, B]) {
    Add.add(p, Neg.neg(p)) = Zero.0[Pair[A, B]]
} by {
    pair_add_group_inverse_law(p)
}

/// Binary pairs of additive groups are additive groups.
instance Pair[A: AddGroup, B: AddGroup]: AddGroup

/// Binary pairs of additive commutative groups are additive commutative groups.
instance Pair[A: AddCommGroup, B: AddCommGroup]: AddCommGroup

/// Componentwise pair multiplication is associative for semigroup factors.
theorem pair_mul_semigroup_law[A: Semigroup, B: Semigroup](
    p: Pair[A, B],
    q: Pair[A, B],
    r: Pair[A, B]
) {
    p * (q * r) = (p * q) * r
} by {
    pair_typeclass_mul_eq_pair_mul(q, r)
    q * r = pair_mul(q, r)
    pair_typeclass_mul_eq_pair_mul(p, q * r)
    p * (q * r) = pair_mul(p, q * r)
    p * (q * r) = pair_mul(p, pair_mul(q, r))

    pair_typeclass_mul_eq_pair_mul(p, q)
    p * q = pair_mul(p, q)
    pair_typeclass_mul_eq_pair_mul(p * q, r)
    (p * q) * r = pair_mul(p * q, r)
    (p * q) * r = pair_mul(pair_mul(p, q), r)

    pair_mul_assoc(p, q, r)
}

/// Componentwise pair multiplication satisfies the exact semigroup typeclass law.
theorem pair_mul_semigroup_instance_law[A: Semigroup, B: Semigroup](
    p: Pair[A, B],
    q: Pair[A, B],
    r: Pair[A, B]
) {
    Mul.mul(p, Mul.mul(q, r)) = Mul.mul(Mul.mul(p, q), r)
} by {
    pair_mul_semigroup_law(p, q, r)
}

/// Binary pairs of semigroups are semigroups.
instance Pair[A: Semigroup, B: Semigroup]: Semigroup

/// Componentwise pair multiplication is commutative for commutative semigroup factors.
theorem pair_mul_comm_semigroup_law[A: CommSemigroup, B: CommSemigroup](
    p: Pair[A, B],
    q: Pair[A, B]
) {
    p * q = q * p
} by {
    pair_typeclass_mul_eq_pair_mul(p, q)
    p * q = pair_mul(p, q)
    pair_typeclass_mul_eq_pair_mul(q, p)
    q * p = pair_mul(q, p)
    pair_mul_comm(p, q)
}

/// Componentwise pair multiplication satisfies the exact commutative-semigroup law.
theorem pair_mul_comm_semigroup_instance_law[A: CommSemigroup, B: CommSemigroup](
    p: Pair[A, B],
    q: Pair[A, B]
) {
    Mul.mul(p, q) = Mul.mul(q, p)
} by {
    pair_mul_comm_semigroup_law(p, q)
}

/// Binary pairs of commutative semigroups are commutative semigroups.
instance Pair[A: CommSemigroup, B: CommSemigroup]: CommSemigroup

/// Componentwise one is a right identity for componentwise pair multiplication.
theorem pair_monoid_right_law[A: Monoid, B: Monoid](p: Pair[A, B]) {
    p * One.1[Pair[A, B]] = p
} by {
    pair_typeclass_one_eq_pair_one[A, B]
    One.1[Pair[A, B]] = pair_one[A, B]
    pair_typeclass_mul_eq_pair_mul(p, One.1[Pair[A, B]])
    p * One.1[Pair[A, B]] = pair_mul(p, One.1[Pair[A, B]])
    p * One.1[Pair[A, B]] = pair_mul(p, pair_one[A, B])
    pair_mul_one_right(p)
}

/// Componentwise one is a left identity for componentwise pair multiplication.
theorem pair_monoid_left_law[A: Monoid, B: Monoid](p: Pair[A, B]) {
    One.1[Pair[A, B]] * p = p
} by {
    pair_typeclass_one_eq_pair_one[A, B]
    One.1[Pair[A, B]] = pair_one[A, B]
    pair_typeclass_mul_eq_pair_mul(One.1[Pair[A, B]], p)
    One.1[Pair[A, B]] * p = pair_mul(One.1[Pair[A, B]], p)
    One.1[Pair[A, B]] * p = pair_mul(pair_one[A, B], p)
    pair_mul_one_left(p)
}

/// Binary pairs of monoids are monoids.
instance Pair[A: Monoid, B: Monoid]: Monoid

/// Binary pairs of commutative monoids are commutative monoids.
instance Pair[A: CommMonoid, B: CommMonoid]: CommMonoid

/// Componentwise multiplication distributes over componentwise addition from the left.
theorem pair_semiring_distrib_left_law[A: Semiring, B: Semiring](
    p: Pair[A, B],
    q: Pair[A, B],
    r: Pair[A, B]
) {
    p * (q + r) = (p * q) + (p * r)
} by {
    pair_typeclass_add_eq_pair_add(q, r)
    q + r = pair_add(q, r)
    pair_typeclass_mul_eq_pair_mul(p, q + r)
    p * (q + r) = pair_mul(p, q + r)
    p * (q + r) = pair_mul(p, pair_add(q, r))

    pair_typeclass_mul_eq_pair_mul(p, q)
    p * q = pair_mul(p, q)
    pair_typeclass_mul_eq_pair_mul(p, r)
    p * r = pair_mul(p, r)
    pair_typeclass_add_eq_pair_add(p * q, p * r)
    (p * q) + (p * r) = pair_add(p * q, p * r)
    (p * q) + (p * r) = pair_add(pair_mul(p, q), pair_mul(p, r))

    pair_mul_add_distrib_left(p, q, r)
}

/// Componentwise multiplication distributes over componentwise addition from the right.
theorem pair_semiring_distrib_right_law[A: Semiring, B: Semiring](
    p: Pair[A, B],
    q: Pair[A, B],
    r: Pair[A, B]
) {
    (p + q) * r = (p * r) + (q * r)
} by {
    pair_typeclass_add_eq_pair_add(p, q)
    p + q = pair_add(p, q)
    pair_typeclass_mul_eq_pair_mul(p + q, r)
    (p + q) * r = pair_mul(p + q, r)
    (p + q) * r = pair_mul(pair_add(p, q), r)

    pair_typeclass_mul_eq_pair_mul(p, r)
    p * r = pair_mul(p, r)
    pair_typeclass_mul_eq_pair_mul(q, r)
    q * r = pair_mul(q, r)
    pair_typeclass_add_eq_pair_add(p * r, q * r)
    (p * r) + (q * r) = pair_add(p * r, q * r)
    (p * r) + (q * r) = pair_add(pair_mul(p, r), pair_mul(q, r))

    pair_mul_add_distrib_right(p, q, r)
}

/// Componentwise zero absorbs pair multiplication on the right.
theorem pair_semiring_mul_zero_left_law[A: Semiring, B: Semiring](p: Pair[A, B]) {
    p * Zero.0[Pair[A, B]] = Zero.0[Pair[A, B]]
} by {
    pair_typeclass_zero_eq_pair_zero[A, B]
    Zero.0[Pair[A, B]] = pair_zero[A, B]
    pair_typeclass_mul_eq_pair_mul(p, Zero.0[Pair[A, B]])
    p * Zero.0[Pair[A, B]] = pair_mul(p, Zero.0[Pair[A, B]])
    p * Zero.0[Pair[A, B]] = pair_mul(p, pair_zero[A, B])
    pair_mul_zero_right(p)
}

/// Componentwise zero absorbs pair multiplication on the left.
theorem pair_semiring_mul_zero_right_law[A: Semiring, B: Semiring](p: Pair[A, B]) {
    Zero.0[Pair[A, B]] * p = Zero.0[Pair[A, B]]
} by {
    pair_typeclass_zero_eq_pair_zero[A, B]
    Zero.0[Pair[A, B]] = pair_zero[A, B]
    pair_typeclass_mul_eq_pair_mul(Zero.0[Pair[A, B]], p)
    Zero.0[Pair[A, B]] * p = pair_mul(Zero.0[Pair[A, B]], p)
    Zero.0[Pair[A, B]] * p = pair_mul(pair_zero[A, B], p)
    pair_mul_zero_left(p)
}

/// Componentwise multiplication satisfies the exact left distributive semiring law.
theorem pair_semiring_distrib_left_instance_law[A: Semiring, B: Semiring](
    p: Pair[A, B],
    q: Pair[A, B],
    r: Pair[A, B]
) {
    Mul.mul(p, Add.add(q, r)) = Add.add(Mul.mul(p, q), Mul.mul(p, r))
} by {
    pair_semiring_distrib_left_law(p, q, r)
}

/// Componentwise multiplication satisfies the exact right distributive semiring law.
theorem pair_semiring_distrib_right_instance_law[A: Semiring, B: Semiring](
    p: Pair[A, B],
    q: Pair[A, B],
    r: Pair[A, B]
) {
    Mul.mul(Add.add(p, q), r) = Add.add(Mul.mul(p, r), Mul.mul(q, r))
} by {
    pair_semiring_distrib_right_law(p, q, r)
}

/// Binary pairs of semirings are semirings.
instance Pair[A: Semiring, B: Semiring]: Semiring

/// Binary pairs of rings are rings.
instance Pair[A: Ring, B: Ring]: Ring

/// Binary pairs of commutative rings are commutative rings.
instance Pair[A: CommRing, B: CommRing]: CommRing
