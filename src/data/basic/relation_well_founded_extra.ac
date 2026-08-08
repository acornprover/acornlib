/// Extra relation-transport and well-founded/noetherian endpoint wrappers.

from data.basic.functions import is_injective_fn, is_surjective_fn, is_bijection_fn,
    bijection_fn_is_injective, bijection_fn_is_surjective, injective_fn_eq,
    surjective_fn_has_preimage, Bijection, bijection_map_is_injective,
    bijection_map_is_surjective
from data.basic.relation_basic import relation_compose, relation_compose_monotone, relation_converse,
    relation_converse_monotone, relation_intersection, relation_intersection_monotone,
    relation_subset, relation_subset_refl, relation_subset_step, relation_union,
    relation_union_monotone
from data.basic.relation_transport import relation_pullback, relation_pullback_monotone,
    relation_pushforward, relation_pushforward_has_preimages, relation_pushforward_intro,
    relation_pushforward_monotone
from algebra.well_founded import has_no_ascending_chain, has_no_descending_chain,
    is_noetherian_relation, is_predecessor_extensional_recursive_step,
    is_successor_closed_predicate, is_well_founded, is_well_founded_recursive_solution,
    relation_pullback_has_no_ascending_chain, relation_pullback_has_no_descending_chain,
    relation_subset_has_no_ascending_chain, relation_subset_has_no_descending_chain,
    relation_subset_is_noetherian_relation, relation_subset_is_well_founded,
    relation_subset_pullback_has_no_ascending_chain,
    relation_subset_pullback_has_no_descending_chain,
    noetherian_relation_has_no_ascending_chain, noetherian_relation_induction,
    well_founded_has_no_descending_chain_exists, well_founded_induction,
    choose_well_founded_recursive_solution, choose_well_founded_recursive_solution_eq_step_at

/// Relation composition is monotone in its left argument.
theorem relation_compose_monotone_left[A, B, C](
    r1: (A, B) -> Bool,
    r2: (A, B) -> Bool,
    s: (B, C) -> Bool
) {
    relation_subset(r1, r2) implies
    relation_subset(relation_compose(r1, s), relation_compose(r2, s))
} by {
    if relation_subset(r1, r2) {
        relation_subset_refl(s)
        relation_subset(s, s)
        relation_compose_monotone(r1, r2, s, s)
        relation_subset(relation_compose(r1, s), relation_compose(r2, s))
    }
}

/// Relation composition is monotone in its right argument.
theorem relation_compose_monotone_right[A, B, C](
    r: (A, B) -> Bool,
    s1: (B, C) -> Bool,
    s2: (B, C) -> Bool
) {
    relation_subset(s1, s2) implies
    relation_subset(relation_compose(r, s1), relation_compose(r, s2))
} by {
    if relation_subset(s1, s2) {
        relation_subset_refl(r)
        relation_subset(r, r)
        relation_compose_monotone(r, r, s1, s2)
        relation_subset(relation_compose(r, s1), relation_compose(r, s2))
    }
}

/// Relation intersection is monotone in its left argument.
theorem relation_intersection_monotone_left[A, B](
    r1: (A, B) -> Bool,
    r2: (A, B) -> Bool,
    s: (A, B) -> Bool
) {
    relation_subset(r1, r2) implies
    relation_subset(relation_intersection(r1, s), relation_intersection(r2, s))
} by {
    if relation_subset(r1, r2) {
        relation_subset_refl(s)
        relation_subset(s, s)
        relation_intersection_monotone(r1, r2, s, s)
        relation_subset(relation_intersection(r1, s), relation_intersection(r2, s))
    }
}

/// Relation intersection is monotone in its right argument.
theorem relation_intersection_monotone_right[A, B](
    r: (A, B) -> Bool,
    s1: (A, B) -> Bool,
    s2: (A, B) -> Bool
) {
    relation_subset(s1, s2) implies
    relation_subset(relation_intersection(r, s1), relation_intersection(r, s2))
} by {
    if relation_subset(s1, s2) {
        relation_subset_refl(r)
        relation_subset(r, r)
        relation_intersection_monotone(r, r, s1, s2)
        relation_subset(relation_intersection(r, s1), relation_intersection(r, s2))
    }
}

/// Relation union is monotone in its left argument.
theorem relation_union_monotone_left[A, B](
    r1: (A, B) -> Bool,
    r2: (A, B) -> Bool,
    s: (A, B) -> Bool
) {
    relation_subset(r1, r2) implies
    relation_subset(relation_union(r1, s), relation_union(r2, s))
} by {
    if relation_subset(r1, r2) {
        relation_subset_refl(s)
        relation_subset(s, s)
        relation_union_monotone(r1, r2, s, s)
        relation_subset(relation_union(r1, s), relation_union(r2, s))
    }
}

/// Relation union is monotone in its right argument.
theorem relation_union_monotone_right[A, B](
    r: (A, B) -> Bool,
    s1: (A, B) -> Bool,
    s2: (A, B) -> Bool
) {
    relation_subset(s1, s2) implies
    relation_subset(relation_union(r, s1), relation_union(r, s2))
} by {
    if relation_subset(s1, s2) {
        relation_subset_refl(r)
        relation_subset(r, r)
        relation_union_monotone(r, r, s1, s2)
        relation_subset(relation_union(r, s1), relation_union(r, s2))
    }
}

/// Converse detects relation inclusion.
theorem relation_converse_subset_iff[A, B](r: (A, B) -> Bool, s: (A, B) -> Bool) {
    relation_subset(relation_converse(r), relation_converse(s)) = relation_subset(r, s)
} by {
    if relation_subset(relation_converse(r), relation_converse(s)) {
        forall(x: A, y: B) {
            if r(x, y) {
                relation_converse(r, y, x)
                relation_subset_step(relation_converse(r), relation_converse(s), y, x)
                relation_converse(s, y, x)
                s(x, y)
            }
        }
        relation_subset(r, s)
    }
    if relation_subset(r, s) {
        relation_converse_monotone(r, s)
        relation_subset(relation_converse(r), relation_converse(s))
    }
}

/// Surjective pullback detects relation inclusion.
theorem relation_pullback_subset_imp_subset_of_surjective[A, B](
    f: A -> B,
    r: (B, B) -> Bool,
    s: (B, B) -> Bool
) {
    is_surjective_fn(f) and
    relation_subset(relation_pullback(f, r), relation_pullback(f, s)) implies
    relation_subset(r, s)
} by {
    if is_surjective_fn(f) and
        relation_subset(relation_pullback(f, r), relation_pullback(f, s)) {
        forall(u: B, v: B) {
            if r(u, v) {
                surjective_fn_has_preimage(f, u)
                let x: A satisfy {
                    f(x) = u
                }
                surjective_fn_has_preimage(f, v)
                let y: A satisfy {
                    f(y) = v
                }
                relation_pullback(f, r, x, y) = r(f(x), f(y))
                relation_pullback(f, r, x, y)
                relation_subset_step(relation_pullback(f, r), relation_pullback(f, s), x, y)
                relation_pullback(f, s, x, y)
                relation_pullback(f, s, x, y) = s(f(x), f(y))
                s(u, v)
            }
        }
    }
}

/// Surjective pullback preserves and reflects relation inclusion.
theorem relation_pullback_subset_iff_of_surjective[A, B](
    f: A -> B,
    r: (B, B) -> Bool,
    s: (B, B) -> Bool
) {
    is_surjective_fn(f) implies
    relation_subset(relation_pullback(f, r), relation_pullback(f, s)) = relation_subset(r, s)
} by {
    if is_surjective_fn(f) {
        if relation_subset(relation_pullback(f, r), relation_pullback(f, s)) {
            relation_pullback_subset_imp_subset_of_surjective(f, r, s)
            relation_subset(r, s)
        }
        if relation_subset(r, s) {
            relation_pullback_monotone(f, r, s)
            relation_subset(relation_pullback(f, r), relation_pullback(f, s))
        }
    }
}

/// Injective pushforward detects relation inclusion.
theorem relation_pushforward_subset_imp_subset_of_injective[A, B](
    f: A -> B,
    r: (A, A) -> Bool,
    s: (A, A) -> Bool
) {
    is_injective_fn(f) and
    relation_subset(relation_pushforward(f, r), relation_pushforward(f, s)) implies
    relation_subset(r, s)
} by {
    if is_injective_fn(f) and
        relation_subset(relation_pushforward(f, r), relation_pushforward(f, s)) {
        forall(x: A, y: A) {
            if r(x, y) {
                relation_pushforward_intro(f, r, x, y)
                relation_pushforward(f, r, f(x), f(y))
                relation_subset_step(
                    relation_pushforward(f, r),
                    relation_pushforward(f, s),
                    f(x),
                    f(y)
                )
                relation_pushforward(f, s, f(x), f(y))
                relation_pushforward_has_preimages(f, s, f(x), f(y))
                let a: A satisfy {
                    exists(b: A) {
                        f(a) = f(x) and f(b) = f(y) and s(a, b)
                    }
                }
                let b: A satisfy {
                    f(a) = f(x) and f(b) = f(y) and s(a, b)
                }
                injective_fn_eq(f, a, x)
                a = x
                injective_fn_eq(f, b, y)
                b = y
                s(x, y)
            }
        }
    }
}

/// Injective pushforward preserves and reflects relation inclusion.
theorem relation_pushforward_subset_iff_of_injective[A, B](
    f: A -> B,
    r: (A, A) -> Bool,
    s: (A, A) -> Bool
) {
    is_injective_fn(f) implies
    relation_subset(relation_pushforward(f, r), relation_pushforward(f, s)) = relation_subset(r, s)
} by {
    if is_injective_fn(f) {
        if relation_subset(relation_pushforward(f, r), relation_pushforward(f, s)) {
            relation_pushforward_subset_imp_subset_of_injective(f, r, s)
            relation_subset(r, s)
        }
        if relation_subset(r, s) {
            relation_pushforward_monotone(f, r, s)
            relation_subset(relation_pushforward(f, r), relation_pushforward(f, s))
        }
    }
}

/// Bijective pullback preserves and reflects relation inclusion.
theorem relation_pullback_subset_iff_of_bijection[A, B](
    f: A -> B,
    r: (B, B) -> Bool,
    s: (B, B) -> Bool
) {
    is_bijection_fn(f) implies
    relation_subset(relation_pullback(f, r), relation_pullback(f, s)) = relation_subset(r, s)
} by {
    if is_bijection_fn(f) {
        bijection_fn_is_surjective(f)
        relation_pullback_subset_iff_of_surjective(f, r, s)
    }
}

/// Bijective pushforward preserves and reflects relation inclusion.
theorem relation_pushforward_subset_iff_of_bijection[A, B](
    f: A -> B,
    r: (A, A) -> Bool,
    s: (A, A) -> Bool
) {
    is_bijection_fn(f) implies
    relation_subset(relation_pushforward(f, r), relation_pushforward(f, s)) = relation_subset(r, s)
} by {
    if is_bijection_fn(f) {
        bijection_fn_is_injective(f)
        relation_pushforward_subset_iff_of_injective(f, r, s)
    }
}

/// Bundled bijective pullback preserves and reflects relation inclusion.
theorem relation_pullback_subset_iff_of_bijection_map[A, B](
    e: Bijection[A, B],
    r: (B, B) -> Bool,
    s: (B, B) -> Bool
) {
    relation_subset(relation_pullback(e.map, r), relation_pullback(e.map, s)) = relation_subset(r, s)
} by {
    bijection_map_is_surjective(e)
    relation_pullback_subset_iff_of_surjective(e.map, r, s)
}

/// Bundled bijective pushforward preserves and reflects relation inclusion.
theorem relation_pushforward_subset_iff_of_bijection_map[A, B](
    e: Bijection[A, B],
    r: (A, A) -> Bool,
    s: (A, A) -> Bool
) {
    relation_subset(relation_pushforward(e.map, r), relation_pushforward(e.map, s)) = relation_subset(r, s)
} by {
    bijection_map_is_injective(e)
    relation_pushforward_subset_iff_of_injective(e.map, r, s)
}

/// True if a predicate is closed under a well-founded predecessor step.
define is_predecessor_closed_predicate[T](r: (T, T) -> Bool, p: T -> Bool) -> Bool {
    forall(x: T) {
        (forall(y: T) { r(y, x) implies p(y) }) implies p(x)
    }
}

/// A predecessor-closed predicate contains an element once it contains all predecessors.
theorem predecessor_closed_predicate_at[T](r: (T, T) -> Bool, p: T -> Bool, x: T) {
    is_predecessor_closed_predicate(r, p) and
    forall(y: T) { r(y, x) implies p(y) } implies p(x)
} by {
    if is_predecessor_closed_predicate(r, p) and
        forall(y: T) { r(y, x) implies p(y) } {
        is_predecessor_closed_predicate(r, p) = forall(z: T) {
            (forall(y: T) { r(y, z) implies p(y) }) implies p(z)
        }
        p(x)
    }
}

/// Well-founded induction from a bundled predecessor-closed predicate hypothesis.
theorem well_founded_induction_of_predecessor_closed[T](r: (T, T) -> Bool, p: T -> Bool) {
    is_well_founded(r) and is_predecessor_closed_predicate(r, p) implies
    forall(x: T) { p(x) }
} by {
    if is_well_founded(r) and is_predecessor_closed_predicate(r, p) {
        forall(x: T) {
            if forall(y: T) { r(y, x) implies p(y) } {
                predecessor_closed_predicate_at(r, p, x)
                p(x)
            }
        }
        well_founded_induction(r, p)
        forall(x: T) { p(x) }
    }
}

/// Well-founded induction at a point from a predecessor-closed predicate hypothesis.
theorem well_founded_induction_at_of_predecessor_closed[T](r: (T, T) -> Bool, p: T -> Bool, x: T) {
    is_well_founded(r) and is_predecessor_closed_predicate(r, p) implies p(x)
} by {
    if is_well_founded(r) and is_predecessor_closed_predicate(r, p) {
        well_founded_induction_of_predecessor_closed(r, p)
        forall(z: T) { p(z) }
        p(x)
    }
}

/// A subrelation of a well-founded relation admits well-founded induction.
theorem relation_subset_well_founded_induction[T](
    r: (T, T) -> Bool,
    s: (T, T) -> Bool,
    p: T -> Bool
) {
    relation_subset(r, s) and is_well_founded(s) and is_predecessor_closed_predicate(r, p)
    implies forall(x: T) { p(x) }
} by {
    if relation_subset(r, s) and is_well_founded(s) and is_predecessor_closed_predicate(r, p) {
        relation_subset_is_well_founded(r, s)
        is_well_founded(r)
        well_founded_induction_of_predecessor_closed(r, p)
        forall(x: T) { p(x) }
    }
}

/// A subrelation of a noetherian relation admits noetherian induction.
theorem relation_subset_noetherian_induction[T](
    r: (T, T) -> Bool,
    s: (T, T) -> Bool,
    p: T -> Bool
) {
    relation_subset(r, s) and is_noetherian_relation(s) and is_successor_closed_predicate(r, p)
    implies forall(x: T) { p(x) }
} by {
    if relation_subset(r, s) and is_noetherian_relation(s) and is_successor_closed_predicate(r, p) {
        relation_subset_is_noetherian_relation(r, s)
        is_noetherian_relation(r)
        noetherian_relation_induction(r, p)
        forall(x: T) { p(x) }
    }
}

/// A subrelation of a well-founded relation has no infinite descending chain.
theorem relation_subset_well_founded_has_no_descending_chain[T](
    r: (T, T) -> Bool,
    s: (T, T) -> Bool
) {
    relation_subset(r, s) and is_well_founded(s) implies has_no_descending_chain(r)
} by {
    if relation_subset(r, s) and is_well_founded(s) {
        well_founded_has_no_descending_chain_exists(s)
        has_no_descending_chain(s)
        relation_subset_has_no_descending_chain(r, s)
        has_no_descending_chain(r)
    }
}

/// A subrelation of a noetherian relation has no infinite ascending chain.
theorem relation_subset_noetherian_has_no_ascending_chain[T](
    r: (T, T) -> Bool,
    s: (T, T) -> Bool
) {
    relation_subset(r, s) and is_noetherian_relation(s) implies has_no_ascending_chain(r)
} by {
    if relation_subset(r, s) and is_noetherian_relation(s) {
        noetherian_relation_has_no_ascending_chain(s)
        has_no_ascending_chain(s)
        relation_subset_has_no_ascending_chain(r, s)
        has_no_ascending_chain(r)
    }
}

/// Pullbacks of well-founded relations have no infinite descending chain.
theorem relation_pullback_well_founded_has_no_descending_chain[A, B](f: A -> B, r: (B, B) -> Bool) {
    is_well_founded(r) implies has_no_descending_chain(relation_pullback(f, r))
} by {
    if is_well_founded(r) {
        well_founded_has_no_descending_chain_exists(r)
        has_no_descending_chain(r)
        relation_pullback_has_no_descending_chain(f, r)
        has_no_descending_chain(relation_pullback(f, r))
    }
}

/// Pullbacks of noetherian relations have no infinite ascending chain.
theorem relation_pullback_noetherian_has_no_ascending_chain[A, B](f: A -> B, r: (B, B) -> Bool) {
    is_noetherian_relation(r) implies has_no_ascending_chain(relation_pullback(f, r))
} by {
    if is_noetherian_relation(r) {
        noetherian_relation_has_no_ascending_chain(r)
        has_no_ascending_chain(r)
        relation_pullback_has_no_ascending_chain(f, r)
        has_no_ascending_chain(relation_pullback(f, r))
    }
}

/// Relations controlled by a well-founded pullback have no infinite descending chain.
theorem relation_subset_pullback_well_founded_has_no_descending_chain_extra[A, B](
    f: A -> B,
    r: (A, A) -> Bool,
    s: (B, B) -> Bool
) {
    relation_subset(r, relation_pullback(f, s)) and is_well_founded(s) implies
    has_no_descending_chain(r)
} by {
    if relation_subset(r, relation_pullback(f, s)) and is_well_founded(s) {
        well_founded_has_no_descending_chain_exists(s)
        has_no_descending_chain(s)
        relation_subset_pullback_has_no_descending_chain(f, r, s)
        has_no_descending_chain(r)
    }
}

/// Relations controlled by a noetherian pullback have no infinite ascending chain.
theorem relation_subset_pullback_noetherian_has_no_ascending_chain[A, B](
    f: A -> B,
    r: (A, A) -> Bool,
    s: (B, B) -> Bool
) {
    relation_subset(r, relation_pullback(f, s)) and is_noetherian_relation(s) implies
    has_no_ascending_chain(r)
} by {
    if relation_subset(r, relation_pullback(f, s)) and is_noetherian_relation(s) {
        noetherian_relation_has_no_ascending_chain(s)
        has_no_ascending_chain(s)
        relation_subset_pullback_has_no_ascending_chain(f, r, s)
        has_no_ascending_chain(r)
    }
}

/// A recursive solution satisfies its recursive equation at a chosen element.
theorem well_founded_recursive_solution_eq_step_at_of_solution[T, U](
    r: (T, T) -> Bool,
    step: (T, T -> U) -> U,
    f: T -> U,
    x: T
) {
    is_well_founded_recursive_solution(r, step, f) implies f(x) = step(x, f)
} by {
    if is_well_founded_recursive_solution(r, step, f) {
        is_well_founded_recursive_solution(r, step, f) = forall(y: T) {
            f(y) = step(y, f)
        }
        f(x) = step(x, f)
    }
}

/// A supplied recursive solution lets the chosen solution satisfy the step equation at a point.
theorem choose_well_founded_recursive_solution_eq_step_at_of_solution[T, U](
    r: (T, T) -> Bool,
    step: (T, T -> U) -> U,
    default: U,
    f: T -> U,
    x: T
) {
    is_well_founded(r) and
    is_predecessor_extensional_recursive_step(r, step) and
    is_well_founded_recursive_solution(r, step, f) implies
    choose_well_founded_recursive_solution(r, step, default)(x) =
    step(x, choose_well_founded_recursive_solution(r, step, default))
} by {
    if is_well_founded(r) and
        is_predecessor_extensional_recursive_step(r, step) and
        is_well_founded_recursive_solution(r, step, f) {
        exists(g: T -> U) {
            g = f and is_well_founded_recursive_solution(r, step, g)
        }
        choose_well_founded_recursive_solution_eq_step_at(r, step, default, x)
        choose_well_founded_recursive_solution(r, step, default)(x) =
        step(x, choose_well_founded_recursive_solution(r, step, default))
    }
}
