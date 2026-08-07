from data.basic.functions import Inhabited, Bijection, compose, compose_bijection,
    compose_bijection_map, inverse_bijection, inverse_bijection_inverse,
    bijection_ext, bijection_eq_map, bijection_map_is_injective,
    bijection_map_is_surjective, compose_cancel_left_of_injective_fn,
    compose_cancel_right_of_surjective_fn, compose_bijection_assoc

/// Equality of bundled bijections is equivalent to equality of their underlying maps.
theorem bijection_map_eq_iff_eq[T, U](e: Bijection[T, U], h: Bijection[T, U]) {
    (e.map = h.map) = (e = h)
} by {
    if e.map = h.map {
        bijection_ext(e, h)
        e = h
    }
    if e = h {
        bijection_eq_map(e, h)
        e.map = h.map
    }
}

/// Equal composite maps with the same left bijection have equal right bundled factors.
theorem compose_bijection_cancel_left_map[T, U, V](
    e: Bijection[U, V], h: Bijection[T, U], k: Bijection[T, U]
) {
    compose_bijection(e, h).map = compose_bijection(e, k).map implies h = k
} by {
    if compose_bijection(e, h).map = compose_bijection(e, k).map {
        compose_bijection_map(e, h)
        compose_bijection_map(e, k)
        bijection_map_is_injective(e)
        compose_cancel_left_of_injective_fn(e.map, h.map, k.map)
        bijection_ext(h, k)
        h = k
    }
}

/// Equal composite maps with the same right bijection have equal left bundled factors.
theorem compose_bijection_cancel_right_map[T, U, V](
    e: Bijection[T, U], h: Bijection[U, V], k: Bijection[U, V]
) {
    compose_bijection(h, e).map = compose_bijection(k, e).map implies h = k
} by {
    if compose_bijection(h, e).map = compose_bijection(k, e).map {
        compose_bijection_map(h, e)
        compose_bijection_map(k, e)
        bijection_map_is_surjective(e)
        compose_cancel_right_of_surjective_fn(e.map, h.map, k.map)
        bijection_ext(h, k)
        h = k
    }
}

/// Equality of composite maps after a fixed left bijection is equivalent to equality of right factors.
theorem compose_bijection_map_eq_iff_right_eq_of_left[T, U, V](
    e: Bijection[U, V], h: Bijection[T, U], k: Bijection[T, U]
) {
    (compose_bijection(e, h).map = compose_bijection(e, k).map) = (h = k)
} by {
    if compose_bijection(e, h).map = compose_bijection(e, k).map {
        compose_bijection_cancel_left_map(e, h, k)
        h = k
    }
    if h = k {
        compose_bijection(e, h).map = compose_bijection(e, k).map
    }
}

/// Equality of composite maps after a fixed right bijection is equivalent to equality of left factors.
theorem compose_bijection_map_eq_iff_left_eq_of_right[T, U, V](
    e: Bijection[T, U], h: Bijection[U, V], k: Bijection[U, V]
) {
    (compose_bijection(h, e).map = compose_bijection(k, e).map) = (h = k)
} by {
    if compose_bijection(h, e).map = compose_bijection(k, e).map {
        compose_bijection_cancel_right_map(e, h, k)
        h = k
    }
    if h = k {
        compose_bijection(h, e).map = compose_bijection(k, e).map
    }
}

/// A left bundled bijection cancels from equality of bundled composites.
theorem compose_bijection_cancel_left[T, U, V](
    e: Bijection[U, V], h: Bijection[T, U], k: Bijection[T, U]
) {
    compose_bijection(e, h) = compose_bijection(e, k) implies h = k
} by {
    if compose_bijection(e, h) = compose_bijection(e, k) {
        bijection_eq_map(compose_bijection(e, h), compose_bijection(e, k))
        compose_bijection_cancel_left_map(e, h, k)
        h = k
    }
}

/// A right bundled bijection cancels from equality of bundled composites.
theorem compose_bijection_cancel_right[T, U, V](
    e: Bijection[T, U], h: Bijection[U, V], k: Bijection[U, V]
) {
    compose_bijection(h, e) = compose_bijection(k, e) implies h = k
} by {
    if compose_bijection(h, e) = compose_bijection(k, e) {
        bijection_eq_map(compose_bijection(h, e), compose_bijection(k, e))
        compose_bijection_cancel_right_map(e, h, k)
        h = k
    }
}

/// Equality of right bundled factors is equivalent to equality after composing with a fixed left bijection.
theorem compose_bijection_eq_iff_right_eq_of_left[T, U, V](
    e: Bijection[U, V], h: Bijection[T, U], k: Bijection[T, U]
) {
    (compose_bijection(e, h) = compose_bijection(e, k)) = (h = k)
} by {
    if compose_bijection(e, h) = compose_bijection(e, k) {
        compose_bijection_cancel_left(e, h, k)
        h = k
    }
    if h = k {
        compose_bijection(e, h) = compose_bijection(e, k)
    }
}

/// Equality of left bundled factors is equivalent to equality after composing with a fixed right bijection.
theorem compose_bijection_eq_iff_left_eq_of_right[T, U, V](
    e: Bijection[T, U], h: Bijection[U, V], k: Bijection[U, V]
) {
    (compose_bijection(h, e) = compose_bijection(k, e)) = (h = k)
} by {
    if compose_bijection(h, e) = compose_bijection(k, e) {
        compose_bijection_cancel_right(e, h, k)
        h = k
    }
    if h = k {
        compose_bijection(h, e) = compose_bijection(k, e)
    }
}

/// Equal inverse bijections come from equal original bijections.
theorem inverse_bijection_eq_imp_eq[T: Inhabited, U: Inhabited](e: Bijection[T, U], h: Bijection[T, U]) {
    inverse_bijection(e) = inverse_bijection(h) implies e = h
} by {
    if inverse_bijection(e) = inverse_bijection(h) {
        inverse_bijection_inverse(e)
        inverse_bijection_inverse(h)
        e = h
    }
}

/// Equal inverse maps come from equal bundled bijections.
theorem inverse_bijection_map_eq_imp_eq[T: Inhabited, U: Inhabited](e: Bijection[T, U], h: Bijection[T, U]) {
    inverse_bijection(e).map = inverse_bijection(h).map implies e = h
} by {
    if inverse_bijection(e).map = inverse_bijection(h).map {
        bijection_ext(inverse_bijection(e), inverse_bijection(h))
        inverse_bijection_eq_imp_eq(e, h)
        e = h
    }
}

/// Taking inverse bijections reflects and preserves equality.
theorem inverse_bijection_eq_iff_eq[T: Inhabited, U: Inhabited](e: Bijection[T, U], h: Bijection[T, U]) {
    (inverse_bijection(e) = inverse_bijection(h)) = (e = h)
} by {
    if inverse_bijection(e) = inverse_bijection(h) {
        inverse_bijection_eq_imp_eq(e, h)
        e = h
    }
    if e = h {
        inverse_bijection(e) = inverse_bijection(h)
    }
}

/// Taking inverse maps reflects and preserves equality of bundled bijections.
theorem inverse_bijection_map_eq_iff_eq[T: Inhabited, U: Inhabited](e: Bijection[T, U], h: Bijection[T, U]) {
    (inverse_bijection(e).map = inverse_bijection(h).map) = (e = h)
} by {
    if inverse_bijection(e).map = inverse_bijection(h).map {
        inverse_bijection_map_eq_imp_eq(e, h)
        e = h
    }
    if e = h {
        inverse_bijection(e).map = inverse_bijection(h).map
    }
}

/// Equal composites with a common middle bijection can cancel that middle factor on the right side.
theorem compose_bijection_cancel_middle_right[T, U, V, W](
    a: Bijection[V, W], b: Bijection[U, V], c: Bijection[T, U], d: Bijection[T, U]
) {
    compose_bijection(a, compose_bijection(b, c)) = compose_bijection(a, compose_bijection(b, d)) implies c = d
} by {
    if compose_bijection(a, compose_bijection(b, c)) = compose_bijection(a, compose_bijection(b, d)) {
        compose_bijection_cancel_left(a, compose_bijection(b, c), compose_bijection(b, d))
        compose_bijection_cancel_left(b, c, d)
        c = d
    }
}

/// Equal composites with a common middle bijection can cancel that middle factor on the left side.
theorem compose_bijection_cancel_middle_left[T, U, V, W](
    a: Bijection[V, W], b: Bijection[U, V], c: Bijection[W, T], d: Bijection[W, T]
) {
    compose_bijection(compose_bijection(c, a), b) = compose_bijection(compose_bijection(d, a), b) implies c = d
} by {
    if compose_bijection(compose_bijection(c, a), b) = compose_bijection(compose_bijection(d, a), b) {
        compose_bijection_assoc(c, a, b)
        compose_bijection_assoc(d, a, b)
        compose_bijection_cancel_right(compose_bijection(a, b), c, d)
        c = d
    }
}
