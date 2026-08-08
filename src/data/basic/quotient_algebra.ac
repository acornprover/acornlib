/// Bridges between kernel-of-homomorphism relations and algebraic congruences.

from algebra.add_monoid import AddMonoid, AddMonoidHom, is_add_monoid_hom, add_monoid_hom_add, add_monoid_hom_zero
from algebra.add_comm_monoid import AddCommMonoid
from algebra.add_group import AddGroup, AddGroupHom, is_add_group_hom, add_group_hom_zero,
    add_group_hom_neg, add_group_hom_add
from algebra.monoid.monoid import Monoid, MonoidHom, monoid_hom_mul, monoid_hom_one, is_monoid_hom
from nat import pow_add, pow_pow, one_pow, pow_zero
from algebra.comm_monoid import CommMonoid
from algebra.group import Group, GroupHom, group_hom_mul, group_hom_inv, group_hom_one, is_group_hom,
    inverse_inverse
from semiring import Semiring
from algebra.semiring_hom import SemiringHom, semiring_hom_add, semiring_hom_mul, semiring_hom_zero, semiring_hom_one
from algebra.ring.ring import Ring
from algebra.ring.ring_hom import RingHom, ring_hom_neg, ring_hom_add, ring_hom_mul, ring_hom_zero,
    ring_hom_one, ring_hom_to_add_group_hom, ring_hom_to_monoid_hom,
    ring_hom_to_add_group_hom_hom, ring_hom_to_monoid_hom_hom
from algebra.add_comm_group import AddCommGroup
from algebra.module.module import Module, module_smul_one, module_smul_add_left, module_smul_add_right, module_smul_assoc,
    module_smul_zero_left, module_smul_zero_right
from algebra.comm_group import CommGroup
from comm_ring import CommRing
from algebra.module.module_hom import is_linear_map
from pair import curry, uncurry
from algebra.module.submodule import linear_map_neg
from algebra.ring.ideal import Ideal, ideal_contains_zero, ideal_contains_add, ideal_contains_neg,
    ideal_contains_mul_left
from data.basic.relation_basic import is_equivalence, is_reflexive, is_symmetric, is_transitive
from data.basic.relation_transport import respects_add, is_add_congruence, respects_binary_op,
    respects_unary_op, is_unary_congruence, is_binary_congruence, respects_add_eq_respects_binary_op,
    add_congruence_eq_binary_congruence, respects_mul, is_mul_congruence,
    respects_mul_eq_respects_binary_op, mul_congruence_eq_binary_congruence
from data.basic.equivalence import kernel_relation, kernel_relation_is_equivalence,
    QuotientRelation, QuotientOver, quotient_over_mk,
    kernel_quotient_relation, kernel_quotient_relation_rel, kernel_relation_respects,
    quotient_over_lift_to, quotient_over_lift_to_projection,
    quotient_over_const, quotient_over_const_projection,
    quotient_unary_respects, quotient_binary_respects,
    quotient_unary_respects_eq_respects_unary_op,
    quotient_binary_respects_eq_respects_binary_op,
    quotient_over_binary_op, quotient_over_unary_op,
    quotient_over_binary_op_projection, quotient_over_binary_op_projection_left,
    quotient_over_binary_op_projection_right,
    quotient_over_binary_op_projection_identity_left,
    quotient_over_binary_op_projection_identity_right,
    quotient_over_unary_op_projection, quotient_over_unary_op_projection_compatible,
    quotient_over_mk_eq_iff_rel
from nat import Nat

/// The kernel of an additive monoid homomorphism preserves addition pairwise.
theorem add_monoid_hom_kernel_step[A: AddMonoid, B: AddMonoid](f: A -> B, x1: A, y1: A, x2: A, y2: A) {
    is_add_monoid_hom(f) and kernel_relation(f, x1, y1) and kernel_relation(f, x2, y2)
    implies kernel_relation(f, x1 + x2, y1 + y2)
} by {
    if is_add_monoid_hom(f) and kernel_relation(f, x1, y1) and kernel_relation(f, x2, y2) {
        is_add_monoid_hom(f) = (f(A.0) = B.0 and forall(a: A, b: A) {
            f(a + b) = f(a) + f(b)
        })
        f(x1) = f(y1)
        f(x2) = f(y2)
        f(x1 + x2) = f(x1) + f(x2)
        f(y1 + y2) = f(y1) + f(y2)
        f(x1 + x2) = f(y1 + y2)
        kernel_relation(f, x1 + x2, y1 + y2)
    }
}

/// The kernel of an additive monoid homomorphism is compatible with addition.
theorem add_monoid_hom_kernel_respects_add[A: AddMonoid, B: AddMonoid](f: A -> B) {
    is_add_monoid_hom(f) implies respects_add(kernel_relation(f))
} by {
    if is_add_monoid_hom(f) {
        respects_add_eq_respects_binary_op(kernel_relation(f))
        respects_add(kernel_relation(f)) = respects_binary_op(kernel_relation(f), A.add)
        forall(x1: A, y1: A, x2: A, y2: A) {
            if kernel_relation(f, x1, y1) and kernel_relation(f, x2, y2) {
                add_monoid_hom_kernel_step(f, x1, y1, x2, y2)
                kernel_relation(f, x1 + x2, y1 + y2)
                kernel_relation(f, A.add(x1, x2), A.add(y1, y2))
            }
        }
        respects_binary_op(kernel_relation(f), A.add)
    }
}

/// The kernel of an additive monoid homomorphism is an additive congruence.
theorem add_monoid_hom_kernel_is_add_congruence[A: AddMonoid, B: AddMonoid](f: A -> B) {
    is_add_monoid_hom(f) implies is_add_congruence(kernel_relation(f))
} by {
    if is_add_monoid_hom(f) {
        kernel_relation_is_equivalence(f)
        add_monoid_hom_kernel_respects_add(f)
        respects_add_eq_respects_binary_op(kernel_relation(f))
        is_binary_congruence(kernel_relation(f), A.add)
        add_congruence_eq_binary_congruence(kernel_relation(f))
    }
}
theorem add_monoid_hom_kernel_zero[A: AddMonoid, B: AddMonoid](f: A -> B, a: A) {
    is_add_monoid_hom(f) and f(a) = B.0 implies kernel_relation(f, a, A.0)
} by {
    if is_add_monoid_hom(f) and f(a) = B.0 {
        is_add_monoid_hom(f) = (f(A.0) = B.0 and forall(a0: A, b0: A) {
            f(a0 + b0) = f(a0) + f(b0)
        })
        f(A.0) = B.0
        f(a) = f(A.0)
        kernel_relation(f, a, A.0)
    }
}

/// The kernel of an additive group homomorphism preserves addition pairwise.
theorem add_group_hom_kernel_step[A: AddGroup, B: AddGroup](f: AddGroupHom[A, B], x1: A, y1: A, x2: A, y2: A) {
    kernel_relation(f.hom, x1, y1) and kernel_relation(f.hom, x2, y2)
    implies kernel_relation(f.hom, x1 + x2, y1 + y2)
} by {
    if kernel_relation(f.hom, x1, y1) and kernel_relation(f.hom, x2, y2) {
        f.hom(x1) = f.hom(y1)
        f.hom(x2) = f.hom(y2)
        add_group_hom_add(f, x1, x2)
        add_group_hom_add(f, y1, y2)
        f.hom(x1 + x2) = f.hom(x1) + f.hom(x2)
        f.hom(y1 + y2) = f.hom(y1) + f.hom(y2)
        f.hom(x1 + x2) = f.hom(y1 + y2)
        kernel_relation(f.hom, x1 + x2, y1 + y2)
    }
}

/// The kernel of an additive group homomorphism is compatible with addition.
theorem add_group_hom_kernel_respects_add[A: AddGroup, B: AddGroup](f: AddGroupHom[A, B]) {
    respects_add(kernel_relation(f.hom))
} by {
    respects_add_eq_respects_binary_op(kernel_relation(f.hom))
    respects_add(kernel_relation(f.hom)) = respects_binary_op(kernel_relation(f.hom), A.add)
    forall(x1: A, y1: A, x2: A, y2: A) {
        if kernel_relation(f.hom, x1, y1) and kernel_relation(f.hom, x2, y2) {
            add_group_hom_kernel_step(f, x1, y1, x2, y2)
            kernel_relation(f.hom, x1 + x2, y1 + y2)
            kernel_relation(f.hom, A.add(x1, x2), A.add(y1, y2))
        }
    }
    respects_binary_op(kernel_relation(f.hom), A.add)
}

/// The kernel of an additive group homomorphism is compatible with negation.
theorem add_group_hom_kernel_respects_neg[A: AddGroup, B: AddGroup](f: AddGroupHom[A, B]) {
    respects_unary_op(kernel_relation(f.hom), A.neg)
} by {
    forall(x: A, y: A) {
        if kernel_relation(f.hom, x, y) {
            f.hom(x) = f.hom(y)
            add_group_hom_neg(f, x)
            add_group_hom_neg(f, y)
            f.hom(-x) = -f.hom(x)
            f.hom(-y) = -f.hom(y)
            f.hom(-x) = f.hom(-y)
            kernel_relation(f.hom, -x, -y)
            kernel_relation(f.hom, A.neg(x), A.neg(y))
        }
    }
}

/// The kernel of an additive group homomorphism is compatible with subtraction.
theorem add_group_hom_kernel_respects_sub[A: AddGroup, B: AddGroup](f: AddGroupHom[A, B]) {
    respects_binary_op(kernel_relation(f.hom), A.sub)
} by {
    forall(x1: A, y1: A, x2: A, y2: A) {
        if kernel_relation(f.hom, x1, y1) and kernel_relation(f.hom, x2, y2) {
            f.hom(x1) = f.hom(y1)
            f.hom(x2) = f.hom(y2)
            add_group_hom_add(f, x1, -x2)
            add_group_hom_add(f, y1, -y2)
            add_group_hom_neg(f, x2)
            add_group_hom_neg(f, y2)
            f.hom(x1 - x2) = f.hom(x1 + -x2)
            f.hom(y1 - y2) = f.hom(y1 + -y2)
            f.hom(x1 + -x2) = f.hom(x1) + f.hom(-x2)
            f.hom(y1 + -y2) = f.hom(y1) + f.hom(-y2)
            f.hom(-x2) = -f.hom(x2)
            f.hom(-y2) = -f.hom(y2)
            f.hom(x1 - x2) = f.hom(y1 - y2)
            kernel_relation(f.hom, x1 - x2, y1 - y2)
            kernel_relation(f.hom, A.sub(x1, x2), A.sub(y1, y2))
        }
    }
}
theorem add_group_hom_kernel_quotient_relation_respects_sub[A: AddGroup, B: AddGroup](f: AddGroupHom[A, B]) {
    quotient_binary_respects(kernel_quotient_relation(f.hom), A.sub)
} by {
    quotient_binary_respects_eq_respects_binary_op(kernel_quotient_relation(f.hom), A.sub)
    kernel_quotient_relation_rel(f.hom)
    add_group_hom_kernel_respects_sub(f)
    respects_binary_op(kernel_relation(f.hom), A.sub)
    respects_binary_op(kernel_quotient_relation(f.hom).rel, A.sub)
}
define add_group_hom_kernel_quotient_sub[A: AddGroup, B: AddGroup](f: AddGroupHom[A, B]) ->
    (QuotientOver[A], QuotientOver[A]) -> QuotientOver[A] {
    quotient_over_binary_op(A.sub)
}
theorem add_group_hom_kernel_quotient_sub_projection[A: AddGroup, B: AddGroup](f: AddGroupHom[A, B], a: A, b: A) {
    add_group_hom_kernel_quotient_sub(f,
        quotient_over_mk(kernel_quotient_relation(f.hom), a),
        quotient_over_mk(kernel_quotient_relation(f.hom), b)
    ) = quotient_over_mk(kernel_quotient_relation(f.hom), a - b)
} by {
    add_group_hom_kernel_quotient_relation_respects_sub(f)
    quotient_over_binary_op_projection(kernel_quotient_relation(f.hom), A.sub, a, b)
}

/// Subtraction in the kernel quotient respects related left representatives.
theorem add_group_hom_kernel_quotient_sub_projection_left[A: AddGroup, B: AddGroup](f: AddGroupHom[A, B],
        a1: A, a2: A, b: A) {
    kernel_relation(f.hom, a1, a2) implies
    add_group_hom_kernel_quotient_sub(f,
        quotient_over_mk(kernel_quotient_relation(f.hom), a1),
        quotient_over_mk(kernel_quotient_relation(f.hom), b)
    ) = add_group_hom_kernel_quotient_sub(f,
        quotient_over_mk(kernel_quotient_relation(f.hom), a2),
        quotient_over_mk(kernel_quotient_relation(f.hom), b)
    )
} by {
    if kernel_relation(f.hom, a1, a2) {
        kernel_quotient_relation_rel(f.hom)
        kernel_quotient_relation(f.hom).rel(a1, a2)
        add_group_hom_kernel_quotient_relation_respects_sub(f)
        quotient_over_binary_op_projection_left(kernel_quotient_relation(f.hom), A.sub, a1, a2, b)
    }
}

/// Subtraction in the kernel quotient respects related right representatives.
theorem add_group_hom_kernel_quotient_sub_projection_right[A: AddGroup, B: AddGroup](f: AddGroupHom[A, B],
        a: A, b1: A, b2: A) {
    kernel_relation(f.hom, b1, b2) implies
    add_group_hom_kernel_quotient_sub(f,
        quotient_over_mk(kernel_quotient_relation(f.hom), a),
        quotient_over_mk(kernel_quotient_relation(f.hom), b1)
    ) = add_group_hom_kernel_quotient_sub(f,
        quotient_over_mk(kernel_quotient_relation(f.hom), a),
        quotient_over_mk(kernel_quotient_relation(f.hom), b2)
    )
} by {
    if kernel_relation(f.hom, b1, b2) {
        kernel_quotient_relation_rel(f.hom)
        kernel_quotient_relation(f.hom).rel(b1, b2)
        add_group_hom_kernel_quotient_relation_respects_sub(f)
        quotient_over_binary_op_projection_right(kernel_quotient_relation(f.hom), A.sub, a, b1, b2)
    }
}

/// The kernel of an additive group homomorphism is an additive congruence.
theorem add_group_hom_kernel_is_add_congruence[A: AddGroup, B: AddGroup](f: AddGroupHom[A, B]) {
    is_add_congruence(kernel_relation(f.hom))
} by {
    kernel_relation_is_equivalence(f.hom)
    add_group_hom_kernel_respects_add(f)
    respects_add_eq_respects_binary_op(kernel_relation(f.hom))
    is_binary_congruence(kernel_relation(f.hom), A.add)
    add_congruence_eq_binary_congruence(kernel_relation(f.hom))
}

/// True if a relation is an additive-group congruence.
define is_add_group_congruence[A: AddGroup](r: (A, A) -> Bool) -> Bool {
    is_add_congruence(r) and is_unary_congruence(r, A.neg)
}

/// The kernel of an additive group homomorphism is compatible with the full additive-group structure.
theorem add_group_hom_kernel_is_add_group_congruence[A: AddGroup, B: AddGroup](f: AddGroupHom[A, B]) {
    is_add_group_congruence(kernel_relation(f.hom))
} by {
    add_group_hom_kernel_is_add_congruence(f)
    kernel_relation_is_equivalence(f.hom)
    add_group_hom_kernel_respects_neg(f)
    is_unary_congruence(kernel_relation(f.hom), A.neg)
}

/// The kernel of a monoid homomorphism preserves multiplication pairwise.
theorem monoid_hom_kernel_step[M: Monoid, N: Monoid](f: MonoidHom[M, N], x1: M, y1: M, x2: M, y2: M) {
    kernel_relation(f.hom, x1, y1) and kernel_relation(f.hom, x2, y2)
    implies kernel_relation(f.hom, x1 * x2, y1 * y2)
} by {
    if kernel_relation(f.hom, x1, y1) and kernel_relation(f.hom, x2, y2) {
        f.hom(x1) = f.hom(y1)
        f.hom(x2) = f.hom(y2)
        monoid_hom_mul(f, x1, x2)
        monoid_hom_mul(f, y1, y2)
        f.hom(x1 * x2) = f.hom(x1) * f.hom(x2)
        f.hom(y1 * y2) = f.hom(y1) * f.hom(y2)
        f.hom(x1 * x2) = f.hom(y1 * y2)
        kernel_relation(f.hom, x1 * x2, y1 * y2)
    }
}

/// The kernel of a monoid homomorphism is compatible with multiplication.
theorem monoid_hom_kernel_respects_mul[M: Monoid, N: Monoid](f: MonoidHom[M, N]) {
    respects_mul(kernel_relation(f.hom))
} by {
    respects_mul_eq_respects_binary_op(kernel_relation(f.hom))
    respects_mul(kernel_relation(f.hom)) = respects_binary_op(kernel_relation(f.hom), M.mul)
    forall(x1: M, y1: M, x2: M, y2: M) {
        if kernel_relation(f.hom, x1, y1) and kernel_relation(f.hom, x2, y2) {
            monoid_hom_kernel_step(f, x1, y1, x2, y2)
            kernel_relation(f.hom, x1 * x2, y1 * y2)
            kernel_relation(f.hom, M.mul(x1, x2), M.mul(y1, y2))
        }
    }
    respects_binary_op(kernel_relation(f.hom), M.mul)
}
theorem monoid_hom_kernel_is_mul_congruence[M: Monoid, N: Monoid](f: MonoidHom[M, N]) {
    is_mul_congruence(kernel_relation(f.hom))
} by {
    kernel_relation_is_equivalence(f.hom)
    monoid_hom_kernel_respects_mul(f)
    respects_mul_eq_respects_binary_op(kernel_relation(f.hom))
    is_binary_congruence(kernel_relation(f.hom), M.mul)
    mul_congruence_eq_binary_congruence(kernel_relation(f.hom))
}

/// True if a relation is a semiring congruence.
define is_semiring_congruence[S: Semiring](r: (S, S) -> Bool) -> Bool {
    is_add_congruence(r) and is_mul_congruence(r)
}

/// The kernel of a semiring homomorphism is compatible with addition.
theorem semiring_hom_kernel_respects_add[S: Semiring, T: Semiring](f: SemiringHom[S, T]) {
    respects_add(kernel_relation(f.hom))
} by {
    respects_add_eq_respects_binary_op(kernel_relation(f.hom))
    respects_add(kernel_relation(f.hom)) = respects_binary_op(kernel_relation(f.hom), S.add)
    forall(x1: S, y1: S, x2: S, y2: S) {
        if kernel_relation(f.hom, x1, y1) and kernel_relation(f.hom, x2, y2) {
            f.hom(x1) = f.hom(y1)
            f.hom(x2) = f.hom(y2)
            semiring_hom_add(f, x1, x2)
            semiring_hom_add(f, y1, y2)
            f.hom(x1 + x2) = f.hom(x1) + f.hom(x2)
            f.hom(y1 + y2) = f.hom(y1) + f.hom(y2)
            f.hom(x1 + x2) = f.hom(y1 + y2)
            kernel_relation(f.hom, x1 + x2, y1 + y2)
            kernel_relation(f.hom, S.add(x1, x2), S.add(y1, y2))
        }
    }
    respects_binary_op(kernel_relation(f.hom), S.add)
}

/// The kernel of a semiring homomorphism is compatible with multiplication.
theorem semiring_hom_kernel_respects_mul[S: Semiring, T: Semiring](f: SemiringHom[S, T]) {
    respects_mul(kernel_relation(f.hom))
} by {
    respects_mul_eq_respects_binary_op(kernel_relation(f.hom))
    respects_mul(kernel_relation(f.hom)) = respects_binary_op(kernel_relation(f.hom), S.mul)
    forall(x1: S, y1: S, x2: S, y2: S) {
        if kernel_relation(f.hom, x1, y1) and kernel_relation(f.hom, x2, y2) {
            f.hom(x1) = f.hom(y1)
            f.hom(x2) = f.hom(y2)
            semiring_hom_mul(f, x1, x2)
            semiring_hom_mul(f, y1, y2)
            f.hom(x1 * x2) = f.hom(x1) * f.hom(x2)
            f.hom(y1 * y2) = f.hom(y1) * f.hom(y2)
            f.hom(x1 * x2) = f.hom(y1 * y2)
            kernel_relation(f.hom, x1 * x2, y1 * y2)
            kernel_relation(f.hom, S.mul(x1, x2), S.mul(y1, y2))
        }
    }
    respects_binary_op(kernel_relation(f.hom), S.mul)
}

/// The kernel of a semiring homomorphism is an additive congruence.
theorem semiring_hom_kernel_is_add_congruence[S: Semiring, T: Semiring](f: SemiringHom[S, T]) {
    is_add_congruence(kernel_relation(f.hom))
} by {
    kernel_relation_is_equivalence(f.hom)
    semiring_hom_kernel_respects_add(f)
    respects_add_eq_respects_binary_op(kernel_relation(f.hom))
    is_binary_congruence(kernel_relation(f.hom), S.add)
    add_congruence_eq_binary_congruence(kernel_relation(f.hom))
}

/// The kernel of a semiring homomorphism is a multiplicative congruence.
theorem semiring_hom_kernel_is_mul_congruence[S: Semiring, T: Semiring](f: SemiringHom[S, T]) {
    is_mul_congruence(kernel_relation(f.hom))
} by {
    kernel_relation_is_equivalence(f.hom)
    semiring_hom_kernel_respects_mul(f)
    respects_mul_eq_respects_binary_op(kernel_relation(f.hom))
    is_binary_congruence(kernel_relation(f.hom), S.mul)
    mul_congruence_eq_binary_congruence(kernel_relation(f.hom))
}

/// The kernel of a semiring homomorphism is a semiring congruence.
theorem semiring_hom_kernel_is_semiring_congruence[S: Semiring, T: Semiring](f: SemiringHom[S, T]) {
    is_semiring_congruence(kernel_relation(f.hom))
} by {
    semiring_hom_kernel_is_add_congruence(f)
    semiring_hom_kernel_is_mul_congruence(f)
}

/// The kernel of a group homomorphism preserves multiplication pairwise.
theorem group_hom_kernel_step[G: Group, H: Group](f: GroupHom[G, H], x1: G, y1: G, x2: G, y2: G) {
    kernel_relation(f.hom, x1, y1) and kernel_relation(f.hom, x2, y2)
    implies kernel_relation(f.hom, x1 * x2, y1 * y2)
} by {
    if kernel_relation(f.hom, x1, y1) and kernel_relation(f.hom, x2, y2) {
        f.hom(x1) = f.hom(y1)
        f.hom(x2) = f.hom(y2)
        group_hom_mul(f, x1, x2)
        group_hom_mul(f, y1, y2)
        f.hom(x1 * x2) = f.hom(x1) * f.hom(x2)
        f.hom(y1 * y2) = f.hom(y1) * f.hom(y2)
        f.hom(x1 * x2) = f.hom(y1 * y2)
        kernel_relation(f.hom, x1 * x2, y1 * y2)
    }
}

/// The kernel of a group homomorphism is compatible with multiplication.
theorem group_hom_kernel_respects_mul[G: Group, H: Group](f: GroupHom[G, H]) {
    respects_mul(kernel_relation(f.hom))
} by {
    respects_mul_eq_respects_binary_op(kernel_relation(f.hom))
    respects_mul(kernel_relation(f.hom)) = respects_binary_op(kernel_relation(f.hom), G.mul)
    forall(x1: G, y1: G, x2: G, y2: G) {
        if kernel_relation(f.hom, x1, y1) and kernel_relation(f.hom, x2, y2) {
            group_hom_kernel_step(f, x1, y1, x2, y2)
            kernel_relation(f.hom, x1 * x2, y1 * y2)
            kernel_relation(f.hom, G.mul(x1, x2), G.mul(y1, y2))
        }
    }
    respects_binary_op(kernel_relation(f.hom), G.mul)
}

/// The kernel of a group homomorphism is compatible with inverses.
theorem group_hom_kernel_respects_inverse[G: Group, H: Group](f: GroupHom[G, H]) {
    respects_unary_op(kernel_relation(f.hom), G.inverse)
} by {
    forall(x: G, y: G) {
        if kernel_relation(f.hom, x, y) {
            f.hom(x) = f.hom(y)
            group_hom_inv(f, x)
            group_hom_inv(f, y)
            f.hom(x.inverse) = f.hom(x).inverse
            f.hom(y.inverse) = f.hom(y).inverse
            f.hom(x.inverse) = f.hom(y.inverse)
            kernel_relation(f.hom, x.inverse, y.inverse)
            kernel_relation(f.hom, G.inverse(x), G.inverse(y))
        }
    }
}
theorem group_hom_kernel_is_mul_congruence[G: Group, H: Group](f: GroupHom[G, H]) {
    is_mul_congruence(kernel_relation(f.hom))
} by {
    kernel_relation_is_equivalence(f.hom)
    group_hom_kernel_respects_mul(f)
    respects_mul_eq_respects_binary_op(kernel_relation(f.hom))
    is_binary_congruence(kernel_relation(f.hom), G.mul)
    mul_congruence_eq_binary_congruence(kernel_relation(f.hom))
}

/// True if a relation is a group congruence.
define is_group_congruence[G: Group](r: (G, G) -> Bool) -> Bool {
    is_mul_congruence(r) and is_unary_congruence(r, G.inverse)
}

/// The kernel of a group homomorphism is compatible with the full group structure.
theorem group_hom_kernel_is_group_congruence[G: Group, H: Group](f: GroupHom[G, H]) {
    is_group_congruence(kernel_relation(f.hom))
} by {
    group_hom_kernel_is_mul_congruence(f)
    kernel_relation_is_equivalence(f.hom)
    group_hom_kernel_respects_inverse(f)
    is_unary_congruence(kernel_relation(f.hom), G.inverse)
}

/// True if a relation is a ring congruence: an equivalence compatible with both addition and multiplication.
define is_ring_congruence[R: Ring](r: (R, R) -> Bool) -> Bool {
    is_add_congruence(r) and is_mul_congruence(r)
}

/// The ideal quotient relation: two elements are related when their difference lies in the ideal.
define ideal_quotient_rel[R: CommRing](i: Ideal[R], a: R, b: R) -> Bool {
    i.contains(a - b)
}

/// Membership in the ideal quotient relation is membership of the difference.
theorem ideal_quotient_rel_eq_contains_sub[R: CommRing](i: Ideal[R], a: R, b: R) {
    ideal_quotient_rel(i, a, b) = i.contains(a - b)
}

/// The ideal quotient relation is reflexive.
theorem ideal_quotient_rel_reflexive_at[R: CommRing](i: Ideal[R], a: R) {
    ideal_quotient_rel(i, a, a)
} by {
    a - a = R.0
    ideal_contains_zero(i)
    i.contains(R.0)
    i.contains(a - a)
}

/// Negating a difference reverses it.
theorem ideal_quotient_neg_sub_eq_swap[R: CommRing](a: R, b: R) {
    -(a - b) = b - a
} by {
    a - b = a + -b
    -(a - b) = -(a + -b)
    -(a + -b) = --b + -a
    --b = b
    --b + -a = b + -a
    b + -a = b - a
}

/// The ideal quotient relation is symmetric.
theorem ideal_quotient_rel_symmetric_at[R: CommRing](i: Ideal[R], a: R, b: R) {
    ideal_quotient_rel(i, a, b) implies ideal_quotient_rel(i, b, a)
} by {
    if ideal_quotient_rel(i, a, b) {
        i.contains(a - b)
        ideal_contains_neg(i, a - b)
        i.contains(-(a - b))
        ideal_quotient_neg_sub_eq_swap(a, b)
        i.contains(b - a)
    }
}

/// Differences compose through an intermediate point.
theorem ideal_quotient_sub_chain[R: CommRing](a: R, b: R, c: R) {
    (a - b) + (b - c) = a - c
} by {
    (a - b) + (b - c) = (a + -b) + (b + -c)
    (a + -b) + (b + -c) = a + (-b + b) + -c
    -b + b = R.0
    a + R.0 + -c = a + -c
    a + -c = a - c
}

/// The ideal quotient relation is transitive.
theorem ideal_quotient_rel_transitive_at[R: CommRing](i: Ideal[R], a: R, b: R, c: R) {
    ideal_quotient_rel(i, a, b) and ideal_quotient_rel(i, b, c) implies ideal_quotient_rel(i, a, c)
} by {
    if ideal_quotient_rel(i, a, b) and ideal_quotient_rel(i, b, c) {
        i.contains(a - b)
        i.contains(b - c)
        ideal_contains_add(i, a - b, b - c)
        i.contains((a - b) + (b - c))
        ideal_quotient_sub_chain(a, b, c)
        i.contains(a - c)
    }
}

/// The ideal quotient relation is reflexive.
theorem ideal_quotient_rel_is_reflexive[R: CommRing](i: Ideal[R]) {
    is_reflexive(ideal_quotient_rel(i))
} by {
    forall(a: R) {
        ideal_quotient_rel_reflexive_at(i, a)
    }
}

/// The ideal quotient relation is symmetric.
theorem ideal_quotient_rel_is_symmetric[R: CommRing](i: Ideal[R]) {
    is_symmetric(ideal_quotient_rel(i))
} by {
    forall(a: R, b: R) {
        if ideal_quotient_rel(i, a, b) {
            ideal_quotient_rel_symmetric_at(i, a, b)
        }
    }
}

/// The ideal quotient relation is transitive.
theorem ideal_quotient_rel_is_transitive[R: CommRing](i: Ideal[R]) {
    is_transitive(ideal_quotient_rel(i))
} by {
    forall(a: R, b: R, c: R) {
        if ideal_quotient_rel(i, a, b) and ideal_quotient_rel(i, b, c) {
            ideal_quotient_rel_transitive_at(i, a, b, c)
            ideal_quotient_rel(i, a, c)
        }
    }
    is_transitive(ideal_quotient_rel(i)) = forall(x: R, y: R, z: R) {
        ideal_quotient_rel(i, x, y) and ideal_quotient_rel(i, y, z) implies ideal_quotient_rel(i, x, z)
    }
}

/// The ideal quotient relation is an equivalence relation.
theorem ideal_quotient_rel_is_equivalence[R: CommRing](i: Ideal[R]) {
    is_equivalence(ideal_quotient_rel(i))
} by {
    ideal_quotient_rel_is_reflexive(i)
    ideal_quotient_rel_is_symmetric(i)
    ideal_quotient_rel_is_transitive(i)
}

/// The sum of related pairs is related.
theorem ideal_quotient_rel_add_step[R: CommRing](i: Ideal[R], a1: R, b1: R, a2: R, b2: R) {
    ideal_quotient_rel(i, a1, b1) and ideal_quotient_rel(i, a2, b2)
    implies ideal_quotient_rel(i, a1 + a2, b1 + b2)
} by {
    if ideal_quotient_rel(i, a1, b1) and ideal_quotient_rel(i, a2, b2) {
        i.contains(a1 - b1)
        i.contains(a2 - b2)
        ideal_contains_add(i, a1 - b1, a2 - b2)
        i.contains((a1 - b1) + (a2 - b2))
        (a1 + a2) - (b1 + b2) = (a1 - b1) + (a2 - b2)
        i.contains((a1 + a2) - (b1 + b2))
    }
}

/// The ideal quotient relation is compatible with addition.
theorem ideal_quotient_rel_respects_add[R: CommRing](i: Ideal[R]) {
    respects_add(ideal_quotient_rel(i))
} by {
    respects_add_eq_respects_binary_op(ideal_quotient_rel(i))
    respects_add(ideal_quotient_rel(i)) = respects_binary_op(ideal_quotient_rel(i), R.add)
    forall(a1: R, b1: R, a2: R, b2: R) {
        if ideal_quotient_rel(i, a1, b1) and ideal_quotient_rel(i, a2, b2) {
            ideal_quotient_rel_add_step(i, a1, b1, a2, b2)
            ideal_quotient_rel(i, a1 + a2, b1 + b2)
            ideal_quotient_rel(i, R.add(a1, a2), R.add(b1, b2))
        }
    }
    respects_binary_op(ideal_quotient_rel(i), R.add) = forall(x1: R, y1: R, x2: R, y2: R) {
        ideal_quotient_rel(i, x1, y1) and ideal_quotient_rel(i, x2, y2)
            implies ideal_quotient_rel(i, R.add(x1, x2), R.add(y1, y2))
    }
    respects_binary_op(ideal_quotient_rel(i), R.add)
}

/// The ideal quotient relation is an additive congruence.
theorem ideal_quotient_rel_is_add_congruence[R: CommRing](i: Ideal[R]) {
    is_add_congruence(ideal_quotient_rel(i))
} by {
    ideal_quotient_rel_is_equivalence(i)
    ideal_quotient_rel_respects_add(i)
    respects_add_eq_respects_binary_op(ideal_quotient_rel(i))
    respects_binary_op(ideal_quotient_rel(i), R.add)
    is_binary_congruence(ideal_quotient_rel(i), R.add) =
        (is_equivalence(ideal_quotient_rel(i)) and respects_binary_op(ideal_quotient_rel(i), R.add))
    is_binary_congruence(ideal_quotient_rel(i), R.add)
    add_congruence_eq_binary_congruence(ideal_quotient_rel(i))
}

/// The difference of products splits into ideal-compatible summands.
theorem ideal_quotient_mul_sub_split[R: CommRing](a1: R, b1: R, a2: R, b2: R) {
    (a1 * a2) - (b1 * b2) = a1 * (a2 - b2) + b2 * (a1 - b1)
} by {
    (a1 * a2) - (b1 * b2) = a1 * a2 + -(b1 * b2)
    a1 * (a2 - b2) = a1 * (a2 + -b2)
    a1 * (a2 + -b2) = a1 * a2 + a1 * -b2
    a1 * -b2 = -(a1 * b2)
    a1 * (a2 - b2) = a1 * a2 + -(a1 * b2)
    b2 * (a1 - b1) = b2 * (a1 + -b1)
    b2 * (a1 + -b1) = b2 * a1 + b2 * -b1
    b2 * a1 = a1 * b2
    b2 * -b1 = -(b2 * b1)
    b2 * b1 = b1 * b2
    b2 * (a1 - b1) = a1 * b2 + -(b1 * b2)
    a1 * (a2 - b2) + b2 * (a1 - b1) =
        (a1 * a2 + -(a1 * b2)) + (a1 * b2 + -(b1 * b2))
    (a1 * a2 + -(a1 * b2)) + (a1 * b2 + -(b1 * b2)) =
        a1 * a2 + (-(a1 * b2) + a1 * b2) + -(b1 * b2)
    -(a1 * b2) + a1 * b2 = R.0
    a1 * a2 + R.0 + -(b1 * b2) = a1 * a2 + -(b1 * b2)
    a1 * (a2 - b2) + b2 * (a1 - b1) = a1 * a2 + -(b1 * b2)
}

/// Multiplication preserves the ideal quotient relation on related pairs.
theorem ideal_quotient_rel_mul_step[R: CommRing](i: Ideal[R], a1: R, b1: R, a2: R, b2: R) {
    ideal_quotient_rel(i, a1, b1) and ideal_quotient_rel(i, a2, b2)
    implies ideal_quotient_rel(i, a1 * a2, b1 * b2)
} by {
    if ideal_quotient_rel(i, a1, b1) and ideal_quotient_rel(i, a2, b2) {
        i.contains(a1 - b1)
        i.contains(a2 - b2)
        ideal_contains_mul_left(i, a1, a2 - b2)
        i.contains(a1 * (a2 - b2))
        ideal_contains_mul_left(i, b2, a1 - b1)
        i.contains(b2 * (a1 - b1))
        ideal_contains_add(i, a1 * (a2 - b2), b2 * (a1 - b1))
        i.contains(a1 * (a2 - b2) + b2 * (a1 - b1))
        ideal_quotient_mul_sub_split(a1, b1, a2, b2)
        i.contains((a1 * a2) - (b1 * b2))
    }
}

/// The ideal quotient relation is compatible with multiplication.
theorem ideal_quotient_rel_respects_mul[R: CommRing](i: Ideal[R]) {
    respects_mul(ideal_quotient_rel(i))
} by {
    respects_mul_eq_respects_binary_op(ideal_quotient_rel(i))
    respects_mul(ideal_quotient_rel(i)) = respects_binary_op(ideal_quotient_rel(i), R.mul)
    forall(a1: R, b1: R, a2: R, b2: R) {
        if ideal_quotient_rel(i, a1, b1) and ideal_quotient_rel(i, a2, b2) {
            ideal_quotient_rel_mul_step(i, a1, b1, a2, b2)
            ideal_quotient_rel(i, a1 * a2, b1 * b2)
            ideal_quotient_rel(i, R.mul(a1, a2), R.mul(b1, b2))
        }
    }
    respects_binary_op(ideal_quotient_rel(i), R.mul) = forall(x1: R, y1: R, x2: R, y2: R) {
        ideal_quotient_rel(i, x1, y1) and ideal_quotient_rel(i, x2, y2)
            implies ideal_quotient_rel(i, R.mul(x1, x2), R.mul(y1, y2))
    }
    respects_binary_op(ideal_quotient_rel(i), R.mul)
}

/// The ideal quotient relation is a multiplicative congruence.
theorem ideal_quotient_rel_is_mul_congruence[R: CommRing](i: Ideal[R]) {
    is_mul_congruence(ideal_quotient_rel(i))
} by {
    ideal_quotient_rel_is_equivalence(i)
    ideal_quotient_rel_respects_mul(i)
    respects_mul_eq_respects_binary_op(ideal_quotient_rel(i))
    respects_binary_op(ideal_quotient_rel(i), R.mul)
    is_binary_congruence(ideal_quotient_rel(i), R.mul) =
        (is_equivalence(ideal_quotient_rel(i)) and respects_binary_op(ideal_quotient_rel(i), R.mul))
    is_binary_congruence(ideal_quotient_rel(i), R.mul)
    mul_congruence_eq_binary_congruence(ideal_quotient_rel(i))
}

/// The ideal quotient relation is a ring congruence.
theorem ideal_quotient_rel_is_ring_congruence[R: CommRing](i: Ideal[R]) {
    is_ring_congruence(ideal_quotient_rel(i))
} by {
    ideal_quotient_rel_is_add_congruence(i)
    ideal_quotient_rel_is_mul_congruence(i)
}

/// The kernel of a ring homomorphism is compatible with addition.
theorem ring_hom_kernel_respects_add[R: Ring, S: Ring](f: RingHom[R, S]) {
    respects_add(kernel_relation(f.hom))
} by {
    let add_f = ring_hom_to_add_group_hom(f)
    ring_hom_to_add_group_hom_hom(f)
    add_f.hom = f.hom
    add_group_hom_kernel_respects_add(add_f)
    respects_add(kernel_relation(add_f.hom))
    respects_add(kernel_relation(f.hom))
}

/// The kernel of a ring homomorphism is compatible with multiplication.
theorem ring_hom_kernel_respects_mul[R: Ring, S: Ring](f: RingHom[R, S]) {
    respects_mul(kernel_relation(f.hom))
} by {
    let mul_f = ring_hom_to_monoid_hom(f)
    ring_hom_to_monoid_hom_hom(f)
    mul_f.hom = f.hom
    monoid_hom_kernel_respects_mul(mul_f)
    respects_mul(kernel_relation(mul_f.hom))
    respects_mul(kernel_relation(f.hom))
}

/// The kernel of a ring homomorphism is compatible with negation.
theorem ring_hom_kernel_respects_neg[R: Ring, S: Ring](f: RingHom[R, S]) {
    respects_unary_op(kernel_relation(f.hom), R.neg)
} by {
    forall(x: R, y: R) {
        if kernel_relation(f.hom, x, y) {
            f.hom(x) = f.hom(y)
            ring_hom_neg(f, x)
            ring_hom_neg(f, y)
            f.hom(-x) = -f.hom(x)
            f.hom(-y) = -f.hom(y)
            f.hom(-x) = f.hom(-y)
            kernel_relation(f.hom, -x, -y)
            kernel_relation(f.hom, R.neg(x), R.neg(y))
        }
    }
}

/// The kernel of a ring homomorphism is compatible with subtraction.
theorem ring_hom_kernel_respects_sub[R: Ring, S: Ring](f: RingHom[R, S]) {
    respects_binary_op(kernel_relation(f.hom), R.sub)
} by {
    forall(x1: R, y1: R, x2: R, y2: R) {
        if kernel_relation(f.hom, x1, y1) and kernel_relation(f.hom, x2, y2) {
            f.hom(x1) = f.hom(y1)
            f.hom(x2) = f.hom(y2)
            ring_hom_neg(f, x2)
            ring_hom_neg(f, y2)
            f.hom(-x2) = -f.hom(x2)
            f.hom(-y2) = -f.hom(y2)
            ring_hom_to_add_group_hom_hom(f)
            let add_f = ring_hom_to_add_group_hom(f)
            add_f.hom = f.hom
            add_group_hom_add(add_f, x1, -x2)
            add_group_hom_add(add_f, y1, -y2)
            f.hom(x1 - x2) = f.hom(x1 + -x2)
            f.hom(y1 - y2) = f.hom(y1 + -y2)
            f.hom(x1 + -x2) = f.hom(x1) + f.hom(-x2)
            f.hom(y1 + -y2) = f.hom(y1) + f.hom(-y2)
            f.hom(x1 - x2) = f.hom(y1 - y2)
            kernel_relation(f.hom, x1 - x2, y1 - y2)
            kernel_relation(f.hom, R.sub(x1, x2), R.sub(y1, y2))
        }
    }
}

/// The kernel of a ring homomorphism is an additive congruence.
theorem ring_hom_kernel_is_add_congruence[R: Ring, S: Ring](f: RingHom[R, S]) {
    is_add_congruence(kernel_relation(f.hom))
} by {
    let add_f = ring_hom_to_add_group_hom(f)
    ring_hom_to_add_group_hom_hom(f)
    add_f.hom = f.hom
    add_group_hom_kernel_is_add_congruence(add_f)
    is_add_congruence(kernel_relation(add_f.hom))
    is_add_congruence(kernel_relation(f.hom))
}

/// The kernel of a ring homomorphism is a multiplicative congruence.
theorem ring_hom_kernel_is_mul_congruence[R: Ring, S: Ring](f: RingHom[R, S]) {
    is_mul_congruence(kernel_relation(f.hom))
} by {
    let mul_f = ring_hom_to_monoid_hom(f)
    ring_hom_to_monoid_hom_hom(f)
    mul_f.hom = f.hom
    monoid_hom_kernel_is_mul_congruence(mul_f)
    is_mul_congruence(kernel_relation(mul_f.hom))
    is_mul_congruence(kernel_relation(f.hom))
}
theorem ring_hom_kernel_quotient_relation_respects_sub[R: Ring, S: Ring](f: RingHom[R, S]) {
    quotient_binary_respects(kernel_quotient_relation(f.hom), R.sub)
} by {
    quotient_binary_respects_eq_respects_binary_op(kernel_quotient_relation(f.hom), R.sub)
    kernel_quotient_relation_rel(f.hom)
    ring_hom_kernel_respects_sub(f)
    respects_binary_op(kernel_relation(f.hom), R.sub)
    respects_binary_op(kernel_quotient_relation(f.hom).rel, R.sub)
}
define ring_hom_kernel_quotient_sub[R: Ring, S: Ring](f: RingHom[R, S]) ->
    (QuotientOver[R], QuotientOver[R]) -> QuotientOver[R] {
    quotient_over_binary_op(R.sub)
}
theorem ring_hom_kernel_quotient_sub_projection[R: Ring, S: Ring](f: RingHom[R, S], a: R, b: R) {
    ring_hom_kernel_quotient_sub(f,
        quotient_over_mk(kernel_quotient_relation(f.hom), a),
        quotient_over_mk(kernel_quotient_relation(f.hom), b)
    ) = quotient_over_mk(kernel_quotient_relation(f.hom), a - b)
} by {
    ring_hom_kernel_quotient_relation_respects_sub(f)
    quotient_over_binary_op_projection(kernel_quotient_relation(f.hom), R.sub, a, b)
}

/// Subtraction in the kernel quotient respects related left representatives.
theorem ring_hom_kernel_quotient_sub_projection_left[R: Ring, S: Ring](f: RingHom[R, S],
        a1: R, a2: R, b: R) {
    kernel_relation(f.hom, a1, a2) implies
    ring_hom_kernel_quotient_sub(f,
        quotient_over_mk(kernel_quotient_relation(f.hom), a1),
        quotient_over_mk(kernel_quotient_relation(f.hom), b)
    ) = ring_hom_kernel_quotient_sub(f,
        quotient_over_mk(kernel_quotient_relation(f.hom), a2),
        quotient_over_mk(kernel_quotient_relation(f.hom), b)
    )
} by {
    if kernel_relation(f.hom, a1, a2) {
        kernel_quotient_relation_rel(f.hom)
        kernel_quotient_relation(f.hom).rel(a1, a2)
        ring_hom_kernel_quotient_relation_respects_sub(f)
        quotient_over_binary_op_projection_left(kernel_quotient_relation(f.hom), R.sub, a1, a2, b)
    }
}

/// Subtraction in the kernel quotient respects related right representatives.
theorem ring_hom_kernel_quotient_sub_projection_right[R: Ring, S: Ring](f: RingHom[R, S],
        a: R, b1: R, b2: R) {
    kernel_relation(f.hom, b1, b2) implies
    ring_hom_kernel_quotient_sub(f,
        quotient_over_mk(kernel_quotient_relation(f.hom), a),
        quotient_over_mk(kernel_quotient_relation(f.hom), b1)
    ) = ring_hom_kernel_quotient_sub(f,
        quotient_over_mk(kernel_quotient_relation(f.hom), a),
        quotient_over_mk(kernel_quotient_relation(f.hom), b2)
    )
} by {
    if kernel_relation(f.hom, b1, b2) {
        kernel_quotient_relation_rel(f.hom)
        kernel_quotient_relation(f.hom).rel(b1, b2)
        ring_hom_kernel_quotient_relation_respects_sub(f)
        quotient_over_binary_op_projection_right(kernel_quotient_relation(f.hom), R.sub, a, b1, b2)
    }
}
theorem ring_hom_kernel_is_ring_congruence[R: Ring, S: Ring](f: RingHom[R, S]) {
    is_ring_congruence(kernel_relation(f.hom))
} by {
    ring_hom_kernel_is_add_congruence(f)
    ring_hom_kernel_is_mul_congruence(f)
}

/// The kernel of a ring homomorphism is an additive-group congruence.
theorem ring_hom_kernel_is_add_group_congruence[R: Ring, S: Ring](f: RingHom[R, S]) {
    is_add_group_congruence(kernel_relation(f.hom))
} by {
    ring_hom_kernel_is_add_congruence(f)
    kernel_relation_is_equivalence(f.hom)
    ring_hom_kernel_respects_neg(f)
    is_unary_congruence(kernel_relation(f.hom), R.neg)
}

/// The kernel of a linear map preserves addition pairwise.
theorem linear_map_kernel_add_step[R: Ring, M: AddCommGroup, N: AddCommGroup](
    src: Module[R, M], dst: Module[R, N], f: M -> N, x1: M, y1: M, x2: M, y2: M
) {
    is_linear_map(src, dst, f) and kernel_relation(f, x1, y1) and kernel_relation(f, x2, y2)
    implies kernel_relation(f, x1 + x2, y1 + y2)
} by {
    if is_linear_map(src, dst, f) and kernel_relation(f, x1, y1) and kernel_relation(f, x2, y2) {
        is_linear_map(src, dst, f) = (
            forall(a: M, b: M) { f(a + b) = f(a) + f(b) }
            and forall(r: R, a: M) { f(src.smul(r, a)) = dst.smul(r, f(a)) }
        )
        f(x1) = f(y1)
        f(x2) = f(y2)
        f(x1 + x2) = f(x1) + f(x2)
        f(y1 + y2) = f(y1) + f(y2)
        f(x1 + x2) = f(y1 + y2)
        kernel_relation(f, x1 + x2, y1 + y2)
    }
}

/// The kernel of a linear map is compatible with addition.
theorem linear_map_kernel_respects_add[R: Ring, M: AddCommGroup, N: AddCommGroup](
    src: Module[R, M], dst: Module[R, N], f: M -> N
) {
    is_linear_map(src, dst, f) implies respects_add(kernel_relation(f))
} by {
    if is_linear_map(src, dst, f) {
        respects_add_eq_respects_binary_op(kernel_relation(f))
        respects_add(kernel_relation(f)) = respects_binary_op(kernel_relation(f), M.add)
        forall(x1: M, y1: M, x2: M, y2: M) {
            if kernel_relation(f, x1, y1) and kernel_relation(f, x2, y2) {
                linear_map_kernel_add_step(src, dst, f, x1, y1, x2, y2)
                kernel_relation(f, x1 + x2, y1 + y2)
                kernel_relation(f, M.add(x1, x2), M.add(y1, y2))
            }
        }
        respects_binary_op(kernel_relation(f), M.add)
    }
}

/// The kernel of a linear map is preserved under scalar multiplication.
theorem linear_map_kernel_smul_step[R: Ring, M: AddCommGroup, N: AddCommGroup](
    src: Module[R, M], dst: Module[R, N], f: M -> N, r: R, x: M, y: M
) {
    is_linear_map(src, dst, f) and kernel_relation(f, x, y)
    implies kernel_relation(f, src.smul(r, x), src.smul(r, y))
} by {
    if is_linear_map(src, dst, f) and kernel_relation(f, x, y) {
        is_linear_map(src, dst, f) = (
            forall(a: M, b: M) { f(a + b) = f(a) + f(b) }
            and forall(r0: R, a: M) { f(src.smul(r0, a)) = dst.smul(r0, f(a)) }
        )
        f(x) = f(y)
        f(src.smul(r, x)) = dst.smul(r, f(x))
        f(src.smul(r, y)) = dst.smul(r, f(y))
        dst.smul(r, f(x)) = dst.smul(r, f(y))
        f(src.smul(r, x)) = f(src.smul(r, y))
        kernel_relation(f, src.smul(r, x), src.smul(r, y))
    }
}
theorem linear_map_kernel_is_add_congruence[R: Ring, M: AddCommGroup, N: AddCommGroup](
    src: Module[R, M], dst: Module[R, N], f: M -> N
) {
    is_linear_map(src, dst, f) implies is_add_congruence(kernel_relation(f))
} by {
    if is_linear_map(src, dst, f) {
        kernel_relation_is_equivalence(f)
        linear_map_kernel_respects_add(src, dst, f)
        respects_add_eq_respects_binary_op(kernel_relation(f))
        is_binary_congruence(kernel_relation(f), M.add)
        add_congruence_eq_binary_congruence(kernel_relation(f))
    }
}

/// The kernel quotient relation of an additive group homomorphism respects addition.
theorem add_group_hom_kernel_quotient_relation_respects_add[A: AddGroup, B: AddGroup](f: AddGroupHom[A, B]) {
    quotient_binary_respects(kernel_quotient_relation(f.hom), A.add)
} by {
    quotient_binary_respects_eq_respects_binary_op(kernel_quotient_relation(f.hom), A.add)
    add_group_hom_kernel_respects_add(f)
    respects_add_eq_respects_binary_op(kernel_relation(f.hom))
    respects_binary_op(kernel_relation(f.hom), A.add)
    kernel_quotient_relation_rel(f.hom)
    kernel_quotient_relation(f.hom).rel = kernel_relation(f.hom)
    respects_binary_op(kernel_quotient_relation(f.hom).rel, A.add)
}

/// The kernel quotient relation of an additive group homomorphism respects negation.
theorem add_group_hom_kernel_quotient_relation_respects_neg[A: AddGroup, B: AddGroup](f: AddGroupHom[A, B]) {
    quotient_unary_respects(kernel_quotient_relation(f.hom), A.neg)
} by {
    quotient_unary_respects_eq_respects_unary_op(kernel_quotient_relation(f.hom), A.neg)
    add_group_hom_kernel_respects_neg(f)
    kernel_quotient_relation_rel(f.hom)
    kernel_quotient_relation(f.hom).rel = kernel_relation(f.hom)
    respects_unary_op(kernel_quotient_relation(f.hom).rel, A.neg)
}

/// Addition on the kernel quotient by an additive group homomorphism.
define add_group_hom_kernel_quotient_add[A: AddGroup, B: AddGroup](f: AddGroupHom[A, B]) ->
    (QuotientOver[A], QuotientOver[A]) -> QuotientOver[A] {
    quotient_over_binary_op(A.add)
}

/// The zero element on the kernel quotient by an additive group homomorphism.
define add_group_hom_kernel_quotient_zero[A: AddGroup, B: AddGroup](f: AddGroupHom[A, B]) -> QuotientOver[A] {
    quotient_over_const(kernel_quotient_relation(f.hom), A.0)
}

/// The zero element on the kernel quotient is the projection of zero.
theorem add_group_hom_kernel_quotient_zero_projection[A: AddGroup, B: AddGroup](f: AddGroupHom[A, B]) {
    add_group_hom_kernel_quotient_zero(f) = quotient_over_mk(kernel_quotient_relation(f.hom), A.0)
} by {
    quotient_over_const_projection(kernel_quotient_relation(f.hom), A.0)
}

/// Addition on the kernel quotient agrees with addition of representatives.
theorem add_group_hom_kernel_quotient_add_projection[A: AddGroup, B: AddGroup](f: AddGroupHom[A, B], a: A, b: A) {
    add_group_hom_kernel_quotient_add(f,
        quotient_over_mk(kernel_quotient_relation(f.hom), a),
        quotient_over_mk(kernel_quotient_relation(f.hom), b)
    ) = quotient_over_mk(kernel_quotient_relation(f.hom), a + b)
} by {
    add_group_hom_kernel_quotient_relation_respects_add(f)
    quotient_over_binary_op_projection(kernel_quotient_relation(f.hom), A.add, a, b)
    A.add(a, b) = a + b
}

/// Addition in the kernel quotient respects related left representatives.
theorem add_group_hom_kernel_quotient_add_projection_left[A: AddGroup, B: AddGroup](f: AddGroupHom[A, B],
        a1: A, a2: A, b: A) {
    kernel_relation(f.hom, a1, a2) implies
    add_group_hom_kernel_quotient_add(f,
        quotient_over_mk(kernel_quotient_relation(f.hom), a1),
        quotient_over_mk(kernel_quotient_relation(f.hom), b)
    ) = add_group_hom_kernel_quotient_add(f,
        quotient_over_mk(kernel_quotient_relation(f.hom), a2),
        quotient_over_mk(kernel_quotient_relation(f.hom), b)
    )
} by {
    if kernel_relation(f.hom, a1, a2) {
        kernel_quotient_relation_rel(f.hom)
        kernel_quotient_relation(f.hom).rel(a1, a2)
        add_group_hom_kernel_quotient_relation_respects_add(f)
        quotient_over_binary_op_projection_left(kernel_quotient_relation(f.hom), A.add, a1, a2, b)
    }
}

/// Addition in the kernel quotient respects related right representatives.
theorem add_group_hom_kernel_quotient_add_projection_right[A: AddGroup, B: AddGroup](f: AddGroupHom[A, B],
        a: A, b1: A, b2: A) {
    kernel_relation(f.hom, b1, b2) implies
    add_group_hom_kernel_quotient_add(f,
        quotient_over_mk(kernel_quotient_relation(f.hom), a),
        quotient_over_mk(kernel_quotient_relation(f.hom), b1)
    ) = add_group_hom_kernel_quotient_add(f,
        quotient_over_mk(kernel_quotient_relation(f.hom), a),
        quotient_over_mk(kernel_quotient_relation(f.hom), b2)
    )
} by {
    if kernel_relation(f.hom, b1, b2) {
        kernel_quotient_relation_rel(f.hom)
        kernel_quotient_relation(f.hom).rel(b1, b2)
        add_group_hom_kernel_quotient_relation_respects_add(f)
        quotient_over_binary_op_projection_right(kernel_quotient_relation(f.hom), A.add, a, b1, b2)
    }
}

/// Zero is a left identity for addition on projected kernel quotient elements.
theorem add_group_hom_kernel_quotient_add_zero_left_projection[A: AddGroup, B: AddGroup](
    f: AddGroupHom[A, B], a: A
) {
    add_group_hom_kernel_quotient_add(f,
        add_group_hom_kernel_quotient_zero(f),
        quotient_over_mk(kernel_quotient_relation(f.hom), a)
    ) = quotient_over_mk(kernel_quotient_relation(f.hom), a)
} by {
    add_group_hom_kernel_quotient_relation_respects_add(f)
    A.0 + a = a
    A.add(A.0, a) = a
    quotient_over_binary_op_projection_identity_left(kernel_quotient_relation(f.hom), A.add, A.0, a)
}

/// Zero is a right identity for addition on projected kernel quotient elements.
theorem add_group_hom_kernel_quotient_add_zero_right_projection[A: AddGroup, B: AddGroup](
    f: AddGroupHom[A, B], a: A
) {
    add_group_hom_kernel_quotient_add(f,
        quotient_over_mk(kernel_quotient_relation(f.hom), a),
        add_group_hom_kernel_quotient_zero(f)
    ) = quotient_over_mk(kernel_quotient_relation(f.hom), a)
} by {
    add_group_hom_kernel_quotient_relation_respects_add(f)
    a + A.0 = a
    A.add(a, A.0) = a
    quotient_over_binary_op_projection_identity_right(kernel_quotient_relation(f.hom), A.add, A.0, a)
}

/// Negation on the kernel quotient by an additive group homomorphism.
define add_group_hom_kernel_quotient_neg[A: AddGroup, B: AddGroup](f: AddGroupHom[A, B]) ->
    QuotientOver[A] -> QuotientOver[A] {
    quotient_over_unary_op(kernel_quotient_relation(f.hom), A.neg)
}

/// Negation on the kernel quotient agrees with negation of representatives.
theorem add_group_hom_kernel_quotient_neg_projection[A: AddGroup, B: AddGroup](f: AddGroupHom[A, B], a: A) {
    add_group_hom_kernel_quotient_neg(f, quotient_over_mk(kernel_quotient_relation(f.hom), a)) =
        quotient_over_mk(kernel_quotient_relation(f.hom), -a)
} by {
    add_group_hom_kernel_quotient_relation_respects_neg(f)
    quotient_over_unary_op_projection(kernel_quotient_relation(f.hom), A.neg, a)
    A.neg(a) = -a
}

/// Negation on the kernel quotient respects related representatives.
theorem add_group_hom_kernel_quotient_neg_projection_compatible[A: AddGroup, B: AddGroup](f: AddGroupHom[A, B],
        a: A, b: A) {
    kernel_relation(f.hom, a, b) implies
    add_group_hom_kernel_quotient_neg(f, quotient_over_mk(kernel_quotient_relation(f.hom), a)) =
        add_group_hom_kernel_quotient_neg(f, quotient_over_mk(kernel_quotient_relation(f.hom), b))
} by {
    if kernel_relation(f.hom, a, b) {
        kernel_quotient_relation_rel(f.hom)
        kernel_quotient_relation(f.hom).rel(a, b)
        add_group_hom_kernel_quotient_relation_respects_neg(f)
        quotient_over_unary_op_projection_compatible(kernel_quotient_relation(f.hom), A.neg, a, b)
    }
}

/// Addition on the additive-group kernel quotient is associative.
theorem add_group_hom_kernel_quotient_add_assoc[A: AddGroup, B: AddGroup](f: AddGroupHom[A, B], a: A, b: A, c: A) {
    add_group_hom_kernel_quotient_add(f,
        add_group_hom_kernel_quotient_add(f,
            quotient_over_mk(kernel_quotient_relation(f.hom), a),
            quotient_over_mk(kernel_quotient_relation(f.hom), b)),
        quotient_over_mk(kernel_quotient_relation(f.hom), c)) =
    add_group_hom_kernel_quotient_add(f,
        quotient_over_mk(kernel_quotient_relation(f.hom), a),
        add_group_hom_kernel_quotient_add(f,
            quotient_over_mk(kernel_quotient_relation(f.hom), b),
            quotient_over_mk(kernel_quotient_relation(f.hom), c)))
} by {
    add_group_hom_kernel_quotient_add_projection(f, a, b)
    add_group_hom_kernel_quotient_add(f,
        quotient_over_mk(kernel_quotient_relation(f.hom), a),
        quotient_over_mk(kernel_quotient_relation(f.hom), b)) =
        quotient_over_mk(kernel_quotient_relation(f.hom), a + b)
    add_group_hom_kernel_quotient_add(f,
        add_group_hom_kernel_quotient_add(f,
            quotient_over_mk(kernel_quotient_relation(f.hom), a),
            quotient_over_mk(kernel_quotient_relation(f.hom), b)),
        quotient_over_mk(kernel_quotient_relation(f.hom), c)) =
    add_group_hom_kernel_quotient_add(f,
        quotient_over_mk(kernel_quotient_relation(f.hom), a + b),
        quotient_over_mk(kernel_quotient_relation(f.hom), c))
    add_group_hom_kernel_quotient_add_projection(f, a + b, c)
    add_group_hom_kernel_quotient_add(f,
        quotient_over_mk(kernel_quotient_relation(f.hom), a + b),
        quotient_over_mk(kernel_quotient_relation(f.hom), c)) =
        quotient_over_mk(kernel_quotient_relation(f.hom), (a + b) + c)
    add_group_hom_kernel_quotient_add_projection(f, b, c)
    add_group_hom_kernel_quotient_add(f,
        quotient_over_mk(kernel_quotient_relation(f.hom), b),
        quotient_over_mk(kernel_quotient_relation(f.hom), c)) =
        quotient_over_mk(kernel_quotient_relation(f.hom), b + c)
    add_group_hom_kernel_quotient_add(f,
        quotient_over_mk(kernel_quotient_relation(f.hom), a),
        add_group_hom_kernel_quotient_add(f,
            quotient_over_mk(kernel_quotient_relation(f.hom), b),
            quotient_over_mk(kernel_quotient_relation(f.hom), c))) =
    add_group_hom_kernel_quotient_add(f,
        quotient_over_mk(kernel_quotient_relation(f.hom), a),
        quotient_over_mk(kernel_quotient_relation(f.hom), b + c))
    add_group_hom_kernel_quotient_add_projection(f, a, b + c)
    add_group_hom_kernel_quotient_add(f,
        quotient_over_mk(kernel_quotient_relation(f.hom), a),
        quotient_over_mk(kernel_quotient_relation(f.hom), b + c)) =
        quotient_over_mk(kernel_quotient_relation(f.hom), a + (b + c))
    (a + b) + c = a + (b + c)
    quotient_over_mk(kernel_quotient_relation(f.hom), (a + b) + c) =
        quotient_over_mk(kernel_quotient_relation(f.hom), a + (b + c))
    add_group_hom_kernel_quotient_add(f,
        add_group_hom_kernel_quotient_add(f,
            quotient_over_mk(kernel_quotient_relation(f.hom), a),
            quotient_over_mk(kernel_quotient_relation(f.hom), b)),
        quotient_over_mk(kernel_quotient_relation(f.hom), c)) =
        quotient_over_mk(kernel_quotient_relation(f.hom), (a + b) + c)
    add_group_hom_kernel_quotient_add(f,
        quotient_over_mk(kernel_quotient_relation(f.hom), a),
        add_group_hom_kernel_quotient_add(f,
            quotient_over_mk(kernel_quotient_relation(f.hom), b),
            quotient_over_mk(kernel_quotient_relation(f.hom), c))) =
        quotient_over_mk(kernel_quotient_relation(f.hom), a + (b + c))
}

/// The quotient zero is a left identity for additive-group quotient addition.
theorem add_group_hom_kernel_quotient_zero_add[A: AddGroup, B: AddGroup](f: AddGroupHom[A, B], a: A) {
    add_group_hom_kernel_quotient_add(f,
        quotient_over_mk(kernel_quotient_relation(f.hom), A.0),
        quotient_over_mk(kernel_quotient_relation(f.hom), a)) =
        quotient_over_mk(kernel_quotient_relation(f.hom), a)
} by {
    add_group_hom_kernel_quotient_add_projection(f, A.0, a)
    A.0 + a = a
}

/// The quotient zero is a right identity for additive-group quotient addition.
theorem add_group_hom_kernel_quotient_add_zero[A: AddGroup, B: AddGroup](f: AddGroupHom[A, B], a: A) {
    add_group_hom_kernel_quotient_add(f,
        quotient_over_mk(kernel_quotient_relation(f.hom), a),
        quotient_over_mk(kernel_quotient_relation(f.hom), A.0)) =
        quotient_over_mk(kernel_quotient_relation(f.hom), a)
} by {
    add_group_hom_kernel_quotient_add_projection(f, a, A.0)
    a + A.0 = a
}

/// The quotient negation is a left inverse for additive-group quotient addition.
theorem add_group_hom_kernel_quotient_neg_add[A: AddGroup, B: AddGroup](f: AddGroupHom[A, B], a: A) {
    add_group_hom_kernel_quotient_add(f,
        add_group_hom_kernel_quotient_neg(f, quotient_over_mk(kernel_quotient_relation(f.hom), a)),
        quotient_over_mk(kernel_quotient_relation(f.hom), a)) =
        quotient_over_mk(kernel_quotient_relation(f.hom), A.0)
} by {
    add_group_hom_kernel_quotient_neg_projection(f, a)
    add_group_hom_kernel_quotient_neg(f, quotient_over_mk(kernel_quotient_relation(f.hom), a)) =
        quotient_over_mk(kernel_quotient_relation(f.hom), -a)
    add_group_hom_kernel_quotient_add(f,
        add_group_hom_kernel_quotient_neg(f, quotient_over_mk(kernel_quotient_relation(f.hom), a)),
        quotient_over_mk(kernel_quotient_relation(f.hom), a)) =
    add_group_hom_kernel_quotient_add(f,
        quotient_over_mk(kernel_quotient_relation(f.hom), -a),
        quotient_over_mk(kernel_quotient_relation(f.hom), a))
    add_group_hom_kernel_quotient_add_projection(f, -a, a)
    add_group_hom_kernel_quotient_add(f,
        quotient_over_mk(kernel_quotient_relation(f.hom), -a),
        quotient_over_mk(kernel_quotient_relation(f.hom), a)) =
        quotient_over_mk(kernel_quotient_relation(f.hom), -a + a)
    -a + a = A.0
    quotient_over_mk(kernel_quotient_relation(f.hom), -a + a) =
        quotient_over_mk(kernel_quotient_relation(f.hom), A.0)
    add_group_hom_kernel_quotient_add(f,
        add_group_hom_kernel_quotient_neg(f, quotient_over_mk(kernel_quotient_relation(f.hom), a)),
        quotient_over_mk(kernel_quotient_relation(f.hom), a)) =
        quotient_over_mk(kernel_quotient_relation(f.hom), A.0)
}

/// The quotient negation is a right inverse for additive-group quotient addition.
theorem add_group_hom_kernel_quotient_add_neg[A: AddGroup, B: AddGroup](f: AddGroupHom[A, B], a: A) {
    add_group_hom_kernel_quotient_add(f,
        quotient_over_mk(kernel_quotient_relation(f.hom), a),
        add_group_hom_kernel_quotient_neg(f, quotient_over_mk(kernel_quotient_relation(f.hom), a))) =
        quotient_over_mk(kernel_quotient_relation(f.hom), A.0)
} by {
    add_group_hom_kernel_quotient_neg_projection(f, a)
    add_group_hom_kernel_quotient_neg(f, quotient_over_mk(kernel_quotient_relation(f.hom), a)) =
        quotient_over_mk(kernel_quotient_relation(f.hom), -a)
    add_group_hom_kernel_quotient_add(f,
        quotient_over_mk(kernel_quotient_relation(f.hom), a),
        add_group_hom_kernel_quotient_neg(f, quotient_over_mk(kernel_quotient_relation(f.hom), a))) =
    add_group_hom_kernel_quotient_add(f,
        quotient_over_mk(kernel_quotient_relation(f.hom), a),
        quotient_over_mk(kernel_quotient_relation(f.hom), -a))
    add_group_hom_kernel_quotient_add_projection(f, a, -a)
    add_group_hom_kernel_quotient_add(f,
        quotient_over_mk(kernel_quotient_relation(f.hom), a),
        quotient_over_mk(kernel_quotient_relation(f.hom), -a)) =
        quotient_over_mk(kernel_quotient_relation(f.hom), a + -a)
    a + -a = A.0
    quotient_over_mk(kernel_quotient_relation(f.hom), a + -a) =
        quotient_over_mk(kernel_quotient_relation(f.hom), A.0)
    add_group_hom_kernel_quotient_add(f,
        quotient_over_mk(kernel_quotient_relation(f.hom), a),
        add_group_hom_kernel_quotient_neg(f, quotient_over_mk(kernel_quotient_relation(f.hom), a))) =
        quotient_over_mk(kernel_quotient_relation(f.hom), A.0)
}

/// Addition on the additive-commutative-group kernel quotient is commutative.
theorem add_group_hom_kernel_quotient_add_comm[A: AddCommGroup, B: AddCommGroup](
    f: AddGroupHom[A, B], a: A, b: A
) {
    add_group_hom_kernel_quotient_add(f,
        quotient_over_mk(kernel_quotient_relation(f.hom), a),
        quotient_over_mk(kernel_quotient_relation(f.hom), b)) =
    add_group_hom_kernel_quotient_add(f,
        quotient_over_mk(kernel_quotient_relation(f.hom), b),
        quotient_over_mk(kernel_quotient_relation(f.hom), a))
} by {
    add_group_hom_kernel_quotient_add_projection(f, a, b)
    add_group_hom_kernel_quotient_add(f,
        quotient_over_mk(kernel_quotient_relation(f.hom), a),
        quotient_over_mk(kernel_quotient_relation(f.hom), b)) =
        quotient_over_mk(kernel_quotient_relation(f.hom), a + b)
    add_group_hom_kernel_quotient_add_projection(f, b, a)
    add_group_hom_kernel_quotient_add(f,
        quotient_over_mk(kernel_quotient_relation(f.hom), b),
        quotient_over_mk(kernel_quotient_relation(f.hom), a)) =
        quotient_over_mk(kernel_quotient_relation(f.hom), b + a)
    a + b = b + a
    quotient_over_mk(kernel_quotient_relation(f.hom), a + b) =
        quotient_over_mk(kernel_quotient_relation(f.hom), b + a)
    add_group_hom_kernel_quotient_add(f,
        quotient_over_mk(kernel_quotient_relation(f.hom), a),
        quotient_over_mk(kernel_quotient_relation(f.hom), b)) =
    add_group_hom_kernel_quotient_add(f,
        quotient_over_mk(kernel_quotient_relation(f.hom), b),
        quotient_over_mk(kernel_quotient_relation(f.hom), a))
}

/// Subtracting quotient zero fixes an additive-group quotient representative.
theorem add_group_hom_kernel_quotient_sub_zero[A: AddGroup, B: AddGroup](
    f: AddGroupHom[A, B], a: A
) {
    add_group_hom_kernel_quotient_sub(f,
        quotient_over_mk(kernel_quotient_relation(f.hom), a),
        quotient_over_mk(kernel_quotient_relation(f.hom), A.0)) =
        quotient_over_mk(kernel_quotient_relation(f.hom), a)
} by {
    add_group_hom_kernel_quotient_sub_projection(f, a, A.0)
    a - A.0 = a + -A.0
    -A.0 = A.0
    a + A.0 = a
}

/// Subtracting a quotient representative from itself gives quotient zero.
theorem add_group_hom_kernel_quotient_sub_self[A: AddGroup, B: AddGroup](
    f: AddGroupHom[A, B], a: A
) {
    add_group_hom_kernel_quotient_sub(f,
        quotient_over_mk(kernel_quotient_relation(f.hom), a),
        quotient_over_mk(kernel_quotient_relation(f.hom), a)) =
        quotient_over_mk(kernel_quotient_relation(f.hom), A.0)
} by {
    add_group_hom_kernel_quotient_sub_projection(f, a, a)
    a - a = a + -a
    a + -a = A.0
}

/// Subtracting a quotient representative from zero is quotient negation.
theorem add_group_hom_kernel_quotient_zero_sub[A: AddGroup, B: AddGroup](
    f: AddGroupHom[A, B], a: A
) {
    add_group_hom_kernel_quotient_sub(f,
        quotient_over_mk(kernel_quotient_relation(f.hom), A.0),
        quotient_over_mk(kernel_quotient_relation(f.hom), a)) =
        add_group_hom_kernel_quotient_neg(f,
            quotient_over_mk(kernel_quotient_relation(f.hom), a))
} by {
    add_group_hom_kernel_quotient_sub_projection(f, A.0, a)
    add_group_hom_kernel_quotient_neg_projection(f, a)
    A.0 - a = A.0 + -a
    A.0 + -a = -a
}

/// Quotient subtraction is addition of quotient negation.
theorem add_group_hom_kernel_quotient_sub_eq_add_neg[A: AddGroup, B: AddGroup](
    f: AddGroupHom[A, B], a: A, b: A
) {
    add_group_hom_kernel_quotient_sub(f,
        quotient_over_mk(kernel_quotient_relation(f.hom), a),
        quotient_over_mk(kernel_quotient_relation(f.hom), b)) =
    add_group_hom_kernel_quotient_add(f,
        quotient_over_mk(kernel_quotient_relation(f.hom), a),
        add_group_hom_kernel_quotient_neg(f,
            quotient_over_mk(kernel_quotient_relation(f.hom), b)))
} by {
    add_group_hom_kernel_quotient_sub_projection(f, a, b)
    add_group_hom_kernel_quotient_neg_projection(f, b)
    add_group_hom_kernel_quotient_add(f,
        quotient_over_mk(kernel_quotient_relation(f.hom), a),
        add_group_hom_kernel_quotient_neg(f,
            quotient_over_mk(kernel_quotient_relation(f.hom), b))) =
    add_group_hom_kernel_quotient_add(f,
        quotient_over_mk(kernel_quotient_relation(f.hom), a),
        quotient_over_mk(kernel_quotient_relation(f.hom), -b))
    add_group_hom_kernel_quotient_add_projection(f, a, -b)
    a - b = a + -b
}

/// A quotient negation cancels a matching left summand.
theorem add_group_hom_kernel_quotient_neg_add_cancel_left[A: AddGroup, B: AddGroup](
    f: AddGroupHom[A, B], a: A, b: A
) {
    add_group_hom_kernel_quotient_add(f,
        add_group_hom_kernel_quotient_neg(f,
            quotient_over_mk(kernel_quotient_relation(f.hom), a)),
        add_group_hom_kernel_quotient_add(f,
            quotient_over_mk(kernel_quotient_relation(f.hom), a),
            quotient_over_mk(kernel_quotient_relation(f.hom), b))) =
        quotient_over_mk(kernel_quotient_relation(f.hom), b)
} by {
    add_group_hom_kernel_quotient_add_projection(f, a, b)
    add_group_hom_kernel_quotient_neg_projection(f, a)
    add_group_hom_kernel_quotient_add(f,
        add_group_hom_kernel_quotient_neg(f,
            quotient_over_mk(kernel_quotient_relation(f.hom), a)),
        add_group_hom_kernel_quotient_add(f,
            quotient_over_mk(kernel_quotient_relation(f.hom), a),
            quotient_over_mk(kernel_quotient_relation(f.hom), b))) =
    add_group_hom_kernel_quotient_add(f,
        quotient_over_mk(kernel_quotient_relation(f.hom), -a),
        quotient_over_mk(kernel_quotient_relation(f.hom), a + b))
    add_group_hom_kernel_quotient_add_projection(f, -a, a + b)
    -a + (a + b) = (-a + a) + b
    -a + a = A.0
    (-a + a) + b = A.0 + b
    A.0 + b = b
    quotient_over_mk(kernel_quotient_relation(f.hom), -a + (a + b)) =
        quotient_over_mk(kernel_quotient_relation(f.hom), b)
}

/// A quotient negation cancels a matching right summand.
theorem add_group_hom_kernel_quotient_add_neg_cancel_right[A: AddGroup, B: AddGroup](
    f: AddGroupHom[A, B], a: A, b: A
) {
    add_group_hom_kernel_quotient_add(f,
        add_group_hom_kernel_quotient_add(f,
            quotient_over_mk(kernel_quotient_relation(f.hom), b),
            quotient_over_mk(kernel_quotient_relation(f.hom), a)),
        add_group_hom_kernel_quotient_neg(f,
            quotient_over_mk(kernel_quotient_relation(f.hom), a))) =
        quotient_over_mk(kernel_quotient_relation(f.hom), b)
} by {
    add_group_hom_kernel_quotient_add_projection(f, b, a)
    add_group_hom_kernel_quotient_neg_projection(f, a)
    add_group_hom_kernel_quotient_add(f,
        add_group_hom_kernel_quotient_add(f,
            quotient_over_mk(kernel_quotient_relation(f.hom), b),
            quotient_over_mk(kernel_quotient_relation(f.hom), a)),
        add_group_hom_kernel_quotient_neg(f,
            quotient_over_mk(kernel_quotient_relation(f.hom), a))) =
    add_group_hom_kernel_quotient_add(f,
        quotient_over_mk(kernel_quotient_relation(f.hom), b + a),
        quotient_over_mk(kernel_quotient_relation(f.hom), -a))
    add_group_hom_kernel_quotient_add_projection(f, b + a, -a)
    (b + a) + -a = b + (a + -a)
    a + -a = A.0
    b + (a + -a) = b + A.0
    b + A.0 = b
    quotient_over_mk(kernel_quotient_relation(f.hom), (b + a) + -a) =
        quotient_over_mk(kernel_quotient_relation(f.hom), b)
}

/// Subtracting the right summand of a quotient sum cancels it.
theorem add_group_hom_kernel_quotient_add_sub_cancel_right[A: AddGroup, B: AddGroup](
    f: AddGroupHom[A, B], a: A, b: A
) {
    add_group_hom_kernel_quotient_sub(f,
        add_group_hom_kernel_quotient_add(f,
            quotient_over_mk(kernel_quotient_relation(f.hom), a),
            quotient_over_mk(kernel_quotient_relation(f.hom), b)),
        quotient_over_mk(kernel_quotient_relation(f.hom), b)) =
        quotient_over_mk(kernel_quotient_relation(f.hom), a)
} by {
    add_group_hom_kernel_quotient_sub_eq_add_neg(f, a + b, b)
    add_group_hom_kernel_quotient_add_projection(f, a, b)
    add_group_hom_kernel_quotient_add_neg_cancel_right(f, b, a)
}

/// Subtracting the left summand of a quotient sum cancels it.
theorem add_group_hom_kernel_quotient_add_sub_cancel_left[A: AddGroup, B: AddGroup](
    f: AddGroupHom[A, B], a: A, b: A
) {
    add_group_hom_kernel_quotient_sub(f,
        add_group_hom_kernel_quotient_add(f,
            quotient_over_mk(kernel_quotient_relation(f.hom), a),
            quotient_over_mk(kernel_quotient_relation(f.hom), b)),
        quotient_over_mk(kernel_quotient_relation(f.hom), a)) =
        quotient_over_mk(kernel_quotient_relation(f.hom), b)
} by {
    add_group_hom_kernel_quotient_add_projection(f, a, b)
    add_group_hom_kernel_quotient_sub_projection(f, a + b, a)
    a + b - a = a + b + -a
    a + b = b + a
    a + b + -a = (b + a) + -a
    (b + a) + -a = b + (a + -a)
    a + -a = A.0
    b + A.0 = b
    quotient_over_mk(kernel_quotient_relation(f.hom), a + b - a) =
        quotient_over_mk(kernel_quotient_relation(f.hom), b)
}

/// The negation of additive-group quotient zero is quotient zero.
theorem add_group_hom_kernel_quotient_neg_zero[A: AddGroup, B: AddGroup](f: AddGroupHom[A, B]) {
    add_group_hom_kernel_quotient_neg(f, add_group_hom_kernel_quotient_zero(f)) =
        add_group_hom_kernel_quotient_zero(f)
} by {
    add_group_hom_kernel_quotient_zero_projection(f)
    add_group_hom_kernel_quotient_neg(f, add_group_hom_kernel_quotient_zero(f)) =
        add_group_hom_kernel_quotient_neg(f, quotient_over_mk(kernel_quotient_relation(f.hom), A.0))
    add_group_hom_kernel_quotient_neg_projection(f, A.0)
    add_group_hom_kernel_quotient_neg(f, quotient_over_mk(kernel_quotient_relation(f.hom), A.0)) =
        quotient_over_mk(kernel_quotient_relation(f.hom), -A.0)
    -A.0 = A.0
    quotient_over_mk(kernel_quotient_relation(f.hom), -A.0) =
        quotient_over_mk(kernel_quotient_relation(f.hom), A.0)
}

/// Negating twice fixes an additive-group quotient representative.
theorem add_group_hom_kernel_quotient_neg_neg[A: AddGroup, B: AddGroup](f: AddGroupHom[A, B], a: A) {
    add_group_hom_kernel_quotient_neg(f,
        add_group_hom_kernel_quotient_neg(f, quotient_over_mk(kernel_quotient_relation(f.hom), a))) =
        quotient_over_mk(kernel_quotient_relation(f.hom), a)
} by {
    add_group_hom_kernel_quotient_neg_projection(f, a)
    add_group_hom_kernel_quotient_neg(f, quotient_over_mk(kernel_quotient_relation(f.hom), a)) =
        quotient_over_mk(kernel_quotient_relation(f.hom), -a)
    add_group_hom_kernel_quotient_neg(f,
        add_group_hom_kernel_quotient_neg(f, quotient_over_mk(kernel_quotient_relation(f.hom), a))) =
        add_group_hom_kernel_quotient_neg(f, quotient_over_mk(kernel_quotient_relation(f.hom), -a))
    add_group_hom_kernel_quotient_neg_projection(f, -a)
    add_group_hom_kernel_quotient_neg(f, quotient_over_mk(kernel_quotient_relation(f.hom), -a)) =
        quotient_over_mk(kernel_quotient_relation(f.hom), -(-a))
    -(-a) = a
    quotient_over_mk(kernel_quotient_relation(f.hom), -(-a)) =
        quotient_over_mk(kernel_quotient_relation(f.hom), a)
}

/// The kernel quotient relation of a group homomorphism respects multiplication.
theorem group_hom_kernel_quotient_relation_respects_mul[G: Group, H: Group](f: GroupHom[G, H]) {
    quotient_binary_respects(kernel_quotient_relation(f.hom), G.mul)
} by {
    quotient_binary_respects_eq_respects_binary_op(kernel_quotient_relation(f.hom), G.mul)
    group_hom_kernel_respects_mul(f)
    respects_mul_eq_respects_binary_op(kernel_relation(f.hom))
    respects_binary_op(kernel_relation(f.hom), G.mul)
    kernel_quotient_relation_rel(f.hom)
    kernel_quotient_relation(f.hom).rel = kernel_relation(f.hom)
    respects_binary_op(kernel_quotient_relation(f.hom).rel, G.mul)
}

/// The kernel quotient relation of a group homomorphism respects inversion.
theorem group_hom_kernel_quotient_relation_respects_inverse[G: Group, H: Group](f: GroupHom[G, H]) {
    quotient_unary_respects(kernel_quotient_relation(f.hom), G.inverse)
} by {
    quotient_unary_respects_eq_respects_unary_op(kernel_quotient_relation(f.hom), G.inverse)
    group_hom_kernel_respects_inverse(f)
    kernel_quotient_relation_rel(f.hom)
    kernel_quotient_relation(f.hom).rel = kernel_relation(f.hom)
    respects_unary_op(kernel_quotient_relation(f.hom).rel, G.inverse)
}

/// Multiplication on the kernel quotient by a group homomorphism.
define group_hom_kernel_quotient_mul[G: Group, H: Group](f: GroupHom[G, H]) ->
    (QuotientOver[G], QuotientOver[G]) -> QuotientOver[G] {
    quotient_over_binary_op(G.mul)
}

/// The identity element on the kernel quotient by a group homomorphism.
define group_hom_kernel_quotient_one[G: Group, H: Group](f: GroupHom[G, H]) -> QuotientOver[G] {
    quotient_over_const(kernel_quotient_relation(f.hom), G.1)
}

/// The identity element on the kernel quotient is the projection of the identity.
theorem group_hom_kernel_quotient_one_projection[G: Group, H: Group](f: GroupHom[G, H]) {
    group_hom_kernel_quotient_one(f) = quotient_over_mk(kernel_quotient_relation(f.hom), G.1)
} by {
    quotient_over_const_projection(kernel_quotient_relation(f.hom), G.1)
}

/// Multiplication on the kernel quotient agrees with multiplication of representatives.
theorem group_hom_kernel_quotient_mul_projection[G: Group, H: Group](f: GroupHom[G, H], a: G, b: G) {
    group_hom_kernel_quotient_mul(f,
        quotient_over_mk(kernel_quotient_relation(f.hom), a),
        quotient_over_mk(kernel_quotient_relation(f.hom), b)
    ) = quotient_over_mk(kernel_quotient_relation(f.hom), a * b)
} by {
    group_hom_kernel_quotient_relation_respects_mul(f)
    quotient_over_binary_op_projection(kernel_quotient_relation(f.hom), G.mul, a, b)
    G.mul(a, b) = a * b
}

/// Multiplication in the kernel quotient respects related left representatives.
theorem group_hom_kernel_quotient_mul_projection_left[G: Group, H: Group](f: GroupHom[G, H],
        a1: G, a2: G, b: G) {
    kernel_relation(f.hom, a1, a2) implies
    group_hom_kernel_quotient_mul(f,
        quotient_over_mk(kernel_quotient_relation(f.hom), a1),
        quotient_over_mk(kernel_quotient_relation(f.hom), b)
    ) = group_hom_kernel_quotient_mul(f,
        quotient_over_mk(kernel_quotient_relation(f.hom), a2),
        quotient_over_mk(kernel_quotient_relation(f.hom), b)
    )
} by {
    if kernel_relation(f.hom, a1, a2) {
        kernel_quotient_relation_rel(f.hom)
        kernel_quotient_relation(f.hom).rel(a1, a2)
        group_hom_kernel_quotient_relation_respects_mul(f)
        quotient_over_binary_op_projection_left(kernel_quotient_relation(f.hom), G.mul, a1, a2, b)
    }
}

/// Multiplication in the kernel quotient respects related right representatives.
theorem group_hom_kernel_quotient_mul_projection_right[G: Group, H: Group](f: GroupHom[G, H],
        a: G, b1: G, b2: G) {
    kernel_relation(f.hom, b1, b2) implies
    group_hom_kernel_quotient_mul(f,
        quotient_over_mk(kernel_quotient_relation(f.hom), a),
        quotient_over_mk(kernel_quotient_relation(f.hom), b1)
    ) = group_hom_kernel_quotient_mul(f,
        quotient_over_mk(kernel_quotient_relation(f.hom), a),
        quotient_over_mk(kernel_quotient_relation(f.hom), b2)
    )
} by {
    if kernel_relation(f.hom, b1, b2) {
        kernel_quotient_relation_rel(f.hom)
        kernel_quotient_relation(f.hom).rel(b1, b2)
        group_hom_kernel_quotient_relation_respects_mul(f)
        quotient_over_binary_op_projection_right(kernel_quotient_relation(f.hom), G.mul, a, b1, b2)
    }
}

/// The identity is a left identity for multiplication on projected kernel quotient elements.
theorem group_hom_kernel_quotient_mul_one_left_projection[G: Group, H: Group](f: GroupHom[G, H], a: G) {
    group_hom_kernel_quotient_mul(f,
        group_hom_kernel_quotient_one(f),
        quotient_over_mk(kernel_quotient_relation(f.hom), a)
    ) = quotient_over_mk(kernel_quotient_relation(f.hom), a)
} by {
    group_hom_kernel_quotient_relation_respects_mul(f)
    G.1 * a = a
    G.mul(G.1, a) = a
    quotient_over_binary_op_projection_identity_left(kernel_quotient_relation(f.hom), G.mul, G.1, a)
}

/// The identity is a right identity for multiplication on projected kernel quotient elements.
theorem group_hom_kernel_quotient_mul_one_right_projection[G: Group, H: Group](f: GroupHom[G, H], a: G) {
    group_hom_kernel_quotient_mul(f,
        quotient_over_mk(kernel_quotient_relation(f.hom), a),
        group_hom_kernel_quotient_one(f)
    ) = quotient_over_mk(kernel_quotient_relation(f.hom), a)
} by {
    group_hom_kernel_quotient_relation_respects_mul(f)
    a * G.1 = a
    G.mul(a, G.1) = a
    quotient_over_binary_op_projection_identity_right(kernel_quotient_relation(f.hom), G.mul, G.1, a)
}

/// Powers on the kernel quotient by a group homomorphism.
define group_hom_kernel_quotient_pow[G: Group, H: Group](
    f: GroupHom[G, H], q: QuotientOver[G], n: Nat
) -> QuotientOver[G] {
    match n {
        Nat.zero {
            group_hom_kernel_quotient_one(f)
        }
        Nat.suc(k) {
            group_hom_kernel_quotient_mul(f, q, group_hom_kernel_quotient_pow(f, q, k))
        }
    }
}

/// The zeroth power on the group kernel quotient is the quotient identity.
theorem group_hom_kernel_quotient_pow_zero_projection[G: Group, H: Group](
    f: GroupHom[G, H], q: QuotientOver[G]
) {
    group_hom_kernel_quotient_pow(f, q, Nat.0) = group_hom_kernel_quotient_one(f)
} by {
}

/// The successor power on the group kernel quotient multiplies by the base.
theorem group_hom_kernel_quotient_pow_suc_projection[G: Group, H: Group](
    f: GroupHom[G, H], q: QuotientOver[G], n: Nat
) {
    group_hom_kernel_quotient_pow(f, q, n.suc) =
        group_hom_kernel_quotient_mul(f, q, group_hom_kernel_quotient_pow(f, q, n))
}

/// The zeroth power on a projected group representative is the projected identity.
theorem group_hom_kernel_quotient_pow_mk_zero[G: Group, H: Group](f: GroupHom[G, H], a: G) {
    group_hom_kernel_quotient_pow(f, quotient_over_mk(kernel_quotient_relation(f.hom), a), Nat.0) =
        quotient_over_mk(kernel_quotient_relation(f.hom), a.pow(Nat.0))
} by {
    group_hom_kernel_quotient_pow_zero_projection(f, quotient_over_mk(kernel_quotient_relation(f.hom), a))
    group_hom_kernel_quotient_one_projection(f)
    a.pow(Nat.0) = G.1
}

/// Powers of projected group representatives agree with projected powers.
theorem group_hom_kernel_quotient_pow_mk[G: Group, H: Group](f: GroupHom[G, H], a: G, n: Nat) {
    group_hom_kernel_quotient_pow(f, quotient_over_mk(kernel_quotient_relation(f.hom), a), n) =
        quotient_over_mk(kernel_quotient_relation(f.hom), a.pow(n))
} by {
    define p(k: Nat) -> Bool {
        group_hom_kernel_quotient_pow(f, quotient_over_mk(kernel_quotient_relation(f.hom), a), k) =
            quotient_over_mk(kernel_quotient_relation(f.hom), a.pow(k))
    }
    group_hom_kernel_quotient_pow_mk_zero(f, a)
    p(Nat.0)
    forall(k: Nat) {
        if p(k) {
            group_hom_kernel_quotient_pow_suc_projection(
                f, quotient_over_mk(kernel_quotient_relation(f.hom), a), k)
            group_hom_kernel_quotient_pow(f,
                quotient_over_mk(kernel_quotient_relation(f.hom), a), k.suc) =
                group_hom_kernel_quotient_mul(f,
                    quotient_over_mk(kernel_quotient_relation(f.hom), a),
                    group_hom_kernel_quotient_pow(f,
                        quotient_over_mk(kernel_quotient_relation(f.hom), a), k))
            group_hom_kernel_quotient_pow(f,
                quotient_over_mk(kernel_quotient_relation(f.hom), a), k) =
                quotient_over_mk(kernel_quotient_relation(f.hom), a.pow(k))
            group_hom_kernel_quotient_mul(f,
                quotient_over_mk(kernel_quotient_relation(f.hom), a),
                group_hom_kernel_quotient_pow(f,
                    quotient_over_mk(kernel_quotient_relation(f.hom), a), k)) =
                group_hom_kernel_quotient_mul(f,
                    quotient_over_mk(kernel_quotient_relation(f.hom), a),
                    quotient_over_mk(kernel_quotient_relation(f.hom), a.pow(k)))
            group_hom_kernel_quotient_mul_projection(f, a, a.pow(k))
            group_hom_kernel_quotient_mul(f,
                quotient_over_mk(kernel_quotient_relation(f.hom), a),
                quotient_over_mk(kernel_quotient_relation(f.hom), a.pow(k))) =
                quotient_over_mk(kernel_quotient_relation(f.hom), a * a.pow(k))
            a.pow(k.suc) = a * a.pow(k)
            quotient_over_mk(kernel_quotient_relation(f.hom), a * a.pow(k)) =
                quotient_over_mk(kernel_quotient_relation(f.hom), a.pow(k.suc))
            p(k.suc)
        }
    }
    p(n)
}

/// Multiplying two projected group-kernel powers adds their exponents.
theorem group_hom_kernel_quotient_pow_add[G: Group, H: Group](
    f: GroupHom[G, H], a: G, n: Nat, m: Nat
) {
    group_hom_kernel_quotient_mul(f,
        group_hom_kernel_quotient_pow(f, quotient_over_mk(kernel_quotient_relation(f.hom), a), n),
        group_hom_kernel_quotient_pow(f, quotient_over_mk(kernel_quotient_relation(f.hom), a), m)) =
    group_hom_kernel_quotient_pow(f, quotient_over_mk(kernel_quotient_relation(f.hom), a), n + m)
} by {
    group_hom_kernel_quotient_pow_mk(f, a, n)
    group_hom_kernel_quotient_pow_mk(f, a, m)
    group_hom_kernel_quotient_mul(f,
        group_hom_kernel_quotient_pow(f, quotient_over_mk(kernel_quotient_relation(f.hom), a), n),
        group_hom_kernel_quotient_pow(f, quotient_over_mk(kernel_quotient_relation(f.hom), a), m)) =
    group_hom_kernel_quotient_mul(f,
        quotient_over_mk(kernel_quotient_relation(f.hom), a.pow(n)),
        quotient_over_mk(kernel_quotient_relation(f.hom), a.pow(m)))
    group_hom_kernel_quotient_mul_projection(f, a.pow(n), a.pow(m))
    group_hom_kernel_quotient_mul(f,
        quotient_over_mk(kernel_quotient_relation(f.hom), a.pow(n)),
        quotient_over_mk(kernel_quotient_relation(f.hom), a.pow(m))) =
        quotient_over_mk(kernel_quotient_relation(f.hom), a.pow(n) * a.pow(m))
    group_hom_kernel_quotient_pow_mk(f, a, n + m)
    group_hom_kernel_quotient_pow(f, quotient_over_mk(kernel_quotient_relation(f.hom), a), n + m) =
        quotient_over_mk(kernel_quotient_relation(f.hom), a.pow(n + m))
    pow_add(a, n, m)
    a.pow(n) * a.pow(m) = a.pow(n + m)
    quotient_over_mk(kernel_quotient_relation(f.hom), a.pow(n) * a.pow(m)) =
        quotient_over_mk(kernel_quotient_relation(f.hom), a.pow(n + m))
}

/// Raising a projected group-kernel power to a power multiplies the exponents.
theorem group_hom_kernel_quotient_pow_pow[G: Group, H: Group](
    f: GroupHom[G, H], a: G, n: Nat, m: Nat
) {
    group_hom_kernel_quotient_pow(f,
        group_hom_kernel_quotient_pow(f, quotient_over_mk(kernel_quotient_relation(f.hom), a), n),
        m) =
    group_hom_kernel_quotient_pow(f, quotient_over_mk(kernel_quotient_relation(f.hom), a), n * m)
} by {
    group_hom_kernel_quotient_pow_mk(f, a, n)
    group_hom_kernel_quotient_pow(f, quotient_over_mk(kernel_quotient_relation(f.hom), a), n) =
        quotient_over_mk(kernel_quotient_relation(f.hom), a.pow(n))
    group_hom_kernel_quotient_pow(f,
        group_hom_kernel_quotient_pow(f, quotient_over_mk(kernel_quotient_relation(f.hom), a), n),
        m) =
    group_hom_kernel_quotient_pow(f,
        quotient_over_mk(kernel_quotient_relation(f.hom), a.pow(n)),
        m)
    group_hom_kernel_quotient_pow_mk(f, a.pow(n), m)
    group_hom_kernel_quotient_pow(f,
        quotient_over_mk(kernel_quotient_relation(f.hom), a.pow(n)),
        m) =
        quotient_over_mk(kernel_quotient_relation(f.hom), a.pow(n).pow(m))
    group_hom_kernel_quotient_pow_mk(f, a, n * m)
    group_hom_kernel_quotient_pow(f, quotient_over_mk(kernel_quotient_relation(f.hom), a), n * m) =
        quotient_over_mk(kernel_quotient_relation(f.hom), a.pow(n * m))
    pow_pow(a, n, m)
    a.pow(n).pow(m) = a.pow(n * m)
    quotient_over_mk(kernel_quotient_relation(f.hom), a.pow(n).pow(m)) =
        quotient_over_mk(kernel_quotient_relation(f.hom), a.pow(n * m))
}

/// Quotient one raised to a natural power is quotient one in a group-kernel quotient.
theorem group_hom_kernel_quotient_one_pow[G: Group, H: Group](f: GroupHom[G, H], n: Nat) {
    group_hom_kernel_quotient_pow(f, group_hom_kernel_quotient_one(f), n) =
        group_hom_kernel_quotient_one(f)
} by {
    group_hom_kernel_quotient_one_projection(f)
    group_hom_kernel_quotient_pow(f, group_hom_kernel_quotient_one(f), n) =
    group_hom_kernel_quotient_pow(f, quotient_over_mk(kernel_quotient_relation(f.hom), G.1), n)
    group_hom_kernel_quotient_pow_mk(f, G.1, n)
    group_hom_kernel_quotient_pow(f, quotient_over_mk(kernel_quotient_relation(f.hom), G.1), n) =
        quotient_over_mk(kernel_quotient_relation(f.hom), G.1.pow(n))
    one_pow[G](n)
    G.1.pow(n) = G.1
    quotient_over_mk(kernel_quotient_relation(f.hom), G.1.pow(n)) =
        quotient_over_mk(kernel_quotient_relation(f.hom), G.1)
}

/// Inversion on the kernel quotient by a group homomorphism.
define group_hom_kernel_quotient_inverse[G: Group, H: Group](f: GroupHom[G, H]) ->
    QuotientOver[G] -> QuotientOver[G] {
    quotient_over_unary_op(kernel_quotient_relation(f.hom), G.inverse)
}

/// Inversion on the kernel quotient agrees with inversion of representatives.
theorem group_hom_kernel_quotient_inverse_projection[G: Group, H: Group](f: GroupHom[G, H], a: G) {
    group_hom_kernel_quotient_inverse(f, quotient_over_mk(kernel_quotient_relation(f.hom), a)) =
        quotient_over_mk(kernel_quotient_relation(f.hom), a.inverse)
} by {
    group_hom_kernel_quotient_relation_respects_inverse(f)
    quotient_over_unary_op_projection(kernel_quotient_relation(f.hom), G.inverse, a)
    G.inverse(a) = a.inverse
}

/// Inversion on the kernel quotient respects related representatives.
theorem group_hom_kernel_quotient_inverse_projection_compatible[G: Group, H: Group](f: GroupHom[G, H],
        a: G, b: G) {
    kernel_relation(f.hom, a, b) implies
    group_hom_kernel_quotient_inverse(f, quotient_over_mk(kernel_quotient_relation(f.hom), a)) =
        group_hom_kernel_quotient_inverse(f, quotient_over_mk(kernel_quotient_relation(f.hom), b))
} by {
    if kernel_relation(f.hom, a, b) {
        kernel_quotient_relation_rel(f.hom)
        kernel_quotient_relation(f.hom).rel(a, b)
        group_hom_kernel_quotient_relation_respects_inverse(f)
        quotient_over_unary_op_projection_compatible(kernel_quotient_relation(f.hom), G.inverse, a, b)
    }
}

/// Inversion twice on a projected group-kernel quotient representative is the original representative.
theorem group_hom_kernel_quotient_inverse_inverse[G: Group, H: Group](f: GroupHom[G, H], a: G) {
    group_hom_kernel_quotient_inverse(f,
        group_hom_kernel_quotient_inverse(f, quotient_over_mk(kernel_quotient_relation(f.hom), a))) =
        quotient_over_mk(kernel_quotient_relation(f.hom), a)
} by {
    group_hom_kernel_quotient_inverse_projection(f, a)
    group_hom_kernel_quotient_inverse(f, quotient_over_mk(kernel_quotient_relation(f.hom), a)) =
        quotient_over_mk(kernel_quotient_relation(f.hom), a.inverse)
    group_hom_kernel_quotient_inverse(f,
        group_hom_kernel_quotient_inverse(f, quotient_over_mk(kernel_quotient_relation(f.hom), a))) =
        group_hom_kernel_quotient_inverse(f, quotient_over_mk(kernel_quotient_relation(f.hom), a.inverse))
    group_hom_kernel_quotient_inverse_projection(f, a.inverse)
    group_hom_kernel_quotient_inverse(f, quotient_over_mk(kernel_quotient_relation(f.hom), a.inverse)) =
        quotient_over_mk(kernel_quotient_relation(f.hom), a.inverse.inverse)
    inverse_inverse(a)
    a.inverse.inverse = a
    quotient_over_mk(kernel_quotient_relation(f.hom), a.inverse.inverse) =
        quotient_over_mk(kernel_quotient_relation(f.hom), a)
    group_hom_kernel_quotient_inverse(f,
        group_hom_kernel_quotient_inverse(f, quotient_over_mk(kernel_quotient_relation(f.hom), a))) =
        quotient_over_mk(kernel_quotient_relation(f.hom), a)
}

/// Multiplication on the group kernel quotient is associative.
theorem group_hom_kernel_quotient_mul_assoc[G: Group, H: Group](f: GroupHom[G, H], a: G, b: G, c: G) {
    group_hom_kernel_quotient_mul(f,
        group_hom_kernel_quotient_mul(f,
            quotient_over_mk(kernel_quotient_relation(f.hom), a),
            quotient_over_mk(kernel_quotient_relation(f.hom), b)),
        quotient_over_mk(kernel_quotient_relation(f.hom), c)) =
    group_hom_kernel_quotient_mul(f,
        quotient_over_mk(kernel_quotient_relation(f.hom), a),
        group_hom_kernel_quotient_mul(f,
            quotient_over_mk(kernel_quotient_relation(f.hom), b),
            quotient_over_mk(kernel_quotient_relation(f.hom), c)))
} by {
    group_hom_kernel_quotient_mul_projection(f, a, b)
    group_hom_kernel_quotient_mul(f,
        quotient_over_mk(kernel_quotient_relation(f.hom), a),
        quotient_over_mk(kernel_quotient_relation(f.hom), b)) =
        quotient_over_mk(kernel_quotient_relation(f.hom), a * b)
    group_hom_kernel_quotient_mul(f,
        group_hom_kernel_quotient_mul(f,
            quotient_over_mk(kernel_quotient_relation(f.hom), a),
            quotient_over_mk(kernel_quotient_relation(f.hom), b)),
        quotient_over_mk(kernel_quotient_relation(f.hom), c)) =
    group_hom_kernel_quotient_mul(f,
        quotient_over_mk(kernel_quotient_relation(f.hom), a * b),
        quotient_over_mk(kernel_quotient_relation(f.hom), c))
    group_hom_kernel_quotient_mul_projection(f, a * b, c)
    group_hom_kernel_quotient_mul(f,
        quotient_over_mk(kernel_quotient_relation(f.hom), a * b),
        quotient_over_mk(kernel_quotient_relation(f.hom), c)) =
        quotient_over_mk(kernel_quotient_relation(f.hom), (a * b) * c)
    group_hom_kernel_quotient_mul_projection(f, b, c)
    group_hom_kernel_quotient_mul(f,
        quotient_over_mk(kernel_quotient_relation(f.hom), b),
        quotient_over_mk(kernel_quotient_relation(f.hom), c)) =
        quotient_over_mk(kernel_quotient_relation(f.hom), b * c)
    group_hom_kernel_quotient_mul(f,
        quotient_over_mk(kernel_quotient_relation(f.hom), a),
        group_hom_kernel_quotient_mul(f,
            quotient_over_mk(kernel_quotient_relation(f.hom), b),
            quotient_over_mk(kernel_quotient_relation(f.hom), c))) =
    group_hom_kernel_quotient_mul(f,
        quotient_over_mk(kernel_quotient_relation(f.hom), a),
        quotient_over_mk(kernel_quotient_relation(f.hom), b * c))
    group_hom_kernel_quotient_mul_projection(f, a, b * c)
    group_hom_kernel_quotient_mul(f,
        quotient_over_mk(kernel_quotient_relation(f.hom), a),
        quotient_over_mk(kernel_quotient_relation(f.hom), b * c)) =
        quotient_over_mk(kernel_quotient_relation(f.hom), a * (b * c))
    (a * b) * c = a * (b * c)
    quotient_over_mk(kernel_quotient_relation(f.hom), (a * b) * c) =
        quotient_over_mk(kernel_quotient_relation(f.hom), a * (b * c))
    group_hom_kernel_quotient_mul(f,
        group_hom_kernel_quotient_mul(f,
            quotient_over_mk(kernel_quotient_relation(f.hom), a),
            quotient_over_mk(kernel_quotient_relation(f.hom), b)),
        quotient_over_mk(kernel_quotient_relation(f.hom), c)) =
        quotient_over_mk(kernel_quotient_relation(f.hom), (a * b) * c)
    group_hom_kernel_quotient_mul(f,
        quotient_over_mk(kernel_quotient_relation(f.hom), a),
        group_hom_kernel_quotient_mul(f,
            quotient_over_mk(kernel_quotient_relation(f.hom), b),
            quotient_over_mk(kernel_quotient_relation(f.hom), c))) =
        quotient_over_mk(kernel_quotient_relation(f.hom), a * (b * c))
}


/// Multiplication on the commutative-group kernel quotient is commutative.
theorem group_hom_kernel_quotient_mul_comm[G: CommGroup, H: CommGroup](f: GroupHom[G, H], a: G, b: G) {
    group_hom_kernel_quotient_mul(f,
        quotient_over_mk(kernel_quotient_relation(f.hom), a),
        quotient_over_mk(kernel_quotient_relation(f.hom), b)) =
    group_hom_kernel_quotient_mul(f,
        quotient_over_mk(kernel_quotient_relation(f.hom), b),
        quotient_over_mk(kernel_quotient_relation(f.hom), a))
} by {
    group_hom_kernel_quotient_mul_projection(f, a, b)
    group_hom_kernel_quotient_mul(f,
        quotient_over_mk(kernel_quotient_relation(f.hom), a),
        quotient_over_mk(kernel_quotient_relation(f.hom), b)) =
        quotient_over_mk(kernel_quotient_relation(f.hom), a * b)
    group_hom_kernel_quotient_mul_projection(f, b, a)
    group_hom_kernel_quotient_mul(f,
        quotient_over_mk(kernel_quotient_relation(f.hom), b),
        quotient_over_mk(kernel_quotient_relation(f.hom), a)) =
        quotient_over_mk(kernel_quotient_relation(f.hom), b * a)
    a * b = b * a
    quotient_over_mk(kernel_quotient_relation(f.hom), a * b) =
        quotient_over_mk(kernel_quotient_relation(f.hom), b * a)
    group_hom_kernel_quotient_mul(f,
        quotient_over_mk(kernel_quotient_relation(f.hom), a),
        quotient_over_mk(kernel_quotient_relation(f.hom), b)) =
    group_hom_kernel_quotient_mul(f,
        quotient_over_mk(kernel_quotient_relation(f.hom), b),
        quotient_over_mk(kernel_quotient_relation(f.hom), a))
}

/// The quotient one is a left identity for group quotient multiplication.
theorem group_hom_kernel_quotient_one_mul[G: Group, H: Group](f: GroupHom[G, H], a: G) {
    group_hom_kernel_quotient_mul(f,
        quotient_over_mk(kernel_quotient_relation(f.hom), G.1),
        quotient_over_mk(kernel_quotient_relation(f.hom), a)) =
        quotient_over_mk(kernel_quotient_relation(f.hom), a)
} by {
    group_hom_kernel_quotient_mul_projection(f, G.1, a)
    G.1 * a = a
}

/// The quotient one is a right identity for group quotient multiplication.
theorem group_hom_kernel_quotient_mul_one[G: Group, H: Group](f: GroupHom[G, H], a: G) {
    group_hom_kernel_quotient_mul(f,
        quotient_over_mk(kernel_quotient_relation(f.hom), a),
        quotient_over_mk(kernel_quotient_relation(f.hom), G.1)) =
        quotient_over_mk(kernel_quotient_relation(f.hom), a)
} by {
    group_hom_kernel_quotient_mul_projection(f, a, G.1)
    a * G.1 = a
}

/// The quotient inverse is a left inverse for group quotient multiplication.
theorem group_hom_kernel_quotient_inverse_left[G: Group, H: Group](f: GroupHom[G, H], a: G) {
    group_hom_kernel_quotient_mul(f,
        group_hom_kernel_quotient_inverse(f, quotient_over_mk(kernel_quotient_relation(f.hom), a)),
        quotient_over_mk(kernel_quotient_relation(f.hom), a)) =
        quotient_over_mk(kernel_quotient_relation(f.hom), G.1)
} by {
    group_hom_kernel_quotient_inverse_projection(f, a)
    group_hom_kernel_quotient_inverse(f, quotient_over_mk(kernel_quotient_relation(f.hom), a)) =
        quotient_over_mk(kernel_quotient_relation(f.hom), a.inverse)
    group_hom_kernel_quotient_mul(f,
        group_hom_kernel_quotient_inverse(f, quotient_over_mk(kernel_quotient_relation(f.hom), a)),
        quotient_over_mk(kernel_quotient_relation(f.hom), a)) =
    group_hom_kernel_quotient_mul(f,
        quotient_over_mk(kernel_quotient_relation(f.hom), a.inverse),
        quotient_over_mk(kernel_quotient_relation(f.hom), a))
    group_hom_kernel_quotient_mul_projection(f, a.inverse, a)
    group_hom_kernel_quotient_mul(f,
        quotient_over_mk(kernel_quotient_relation(f.hom), a.inverse),
        quotient_over_mk(kernel_quotient_relation(f.hom), a)) =
        quotient_over_mk(kernel_quotient_relation(f.hom), a.inverse * a)
    a.inverse * a = G.1
    quotient_over_mk(kernel_quotient_relation(f.hom), a.inverse * a) =
        quotient_over_mk(kernel_quotient_relation(f.hom), G.1)
    group_hom_kernel_quotient_mul(f,
        group_hom_kernel_quotient_inverse(f, quotient_over_mk(kernel_quotient_relation(f.hom), a)),
        quotient_over_mk(kernel_quotient_relation(f.hom), a)) =
        quotient_over_mk(kernel_quotient_relation(f.hom), G.1)
}

/// The quotient inverse is a right inverse for group quotient multiplication.
theorem group_hom_kernel_quotient_inverse_right[G: Group, H: Group](f: GroupHom[G, H], a: G) {
    group_hom_kernel_quotient_mul(f,
        quotient_over_mk(kernel_quotient_relation(f.hom), a),
        group_hom_kernel_quotient_inverse(f, quotient_over_mk(kernel_quotient_relation(f.hom), a))) =
        quotient_over_mk(kernel_quotient_relation(f.hom), G.1)
} by {
    group_hom_kernel_quotient_inverse_projection(f, a)
    group_hom_kernel_quotient_inverse(f, quotient_over_mk(kernel_quotient_relation(f.hom), a)) =
        quotient_over_mk(kernel_quotient_relation(f.hom), a.inverse)
    group_hom_kernel_quotient_mul(f,
        quotient_over_mk(kernel_quotient_relation(f.hom), a),
        group_hom_kernel_quotient_inverse(f, quotient_over_mk(kernel_quotient_relation(f.hom), a))) =
    group_hom_kernel_quotient_mul(f,
        quotient_over_mk(kernel_quotient_relation(f.hom), a),
        quotient_over_mk(kernel_quotient_relation(f.hom), a.inverse))
    group_hom_kernel_quotient_mul_projection(f, a, a.inverse)
    group_hom_kernel_quotient_mul(f,
        quotient_over_mk(kernel_quotient_relation(f.hom), a),
        quotient_over_mk(kernel_quotient_relation(f.hom), a.inverse)) =
        quotient_over_mk(kernel_quotient_relation(f.hom), a * a.inverse)
    a * a.inverse = G.1
    quotient_over_mk(kernel_quotient_relation(f.hom), a * a.inverse) =
        quotient_over_mk(kernel_quotient_relation(f.hom), G.1)
    group_hom_kernel_quotient_mul(f,
        quotient_over_mk(kernel_quotient_relation(f.hom), a),
        group_hom_kernel_quotient_inverse(f, quotient_over_mk(kernel_quotient_relation(f.hom), a))) =
        quotient_over_mk(kernel_quotient_relation(f.hom), G.1)
}

/// Multiplication by a quotient inverse cancels on the left.
theorem group_hom_kernel_quotient_inverse_mul_cancel_left[G: Group, H: Group](
    f: GroupHom[G, H], a: G, b: G
) {
    group_hom_kernel_quotient_mul(f,
        group_hom_kernel_quotient_inverse(f, quotient_over_mk(kernel_quotient_relation(f.hom), a)),
        group_hom_kernel_quotient_mul(f,
            quotient_over_mk(kernel_quotient_relation(f.hom), a),
            quotient_over_mk(kernel_quotient_relation(f.hom), b))) =
        quotient_over_mk(kernel_quotient_relation(f.hom), b)
} by {
    group_hom_kernel_quotient_mul_projection(f, a, b)
    group_hom_kernel_quotient_mul(f,
        quotient_over_mk(kernel_quotient_relation(f.hom), a),
        quotient_over_mk(kernel_quotient_relation(f.hom), b)) =
        quotient_over_mk(kernel_quotient_relation(f.hom), a * b)
    group_hom_kernel_quotient_inverse_projection(f, a)
    group_hom_kernel_quotient_inverse(f, quotient_over_mk(kernel_quotient_relation(f.hom), a)) =
        quotient_over_mk(kernel_quotient_relation(f.hom), a.inverse)
    group_hom_kernel_quotient_mul(f,
        group_hom_kernel_quotient_inverse(f, quotient_over_mk(kernel_quotient_relation(f.hom), a)),
        group_hom_kernel_quotient_mul(f,
            quotient_over_mk(kernel_quotient_relation(f.hom), a),
            quotient_over_mk(kernel_quotient_relation(f.hom), b))) =
    group_hom_kernel_quotient_mul(f,
        quotient_over_mk(kernel_quotient_relation(f.hom), a.inverse),
        quotient_over_mk(kernel_quotient_relation(f.hom), a * b))
    group_hom_kernel_quotient_mul_projection(f, a.inverse, a * b)
    group_hom_kernel_quotient_mul(f,
        quotient_over_mk(kernel_quotient_relation(f.hom), a.inverse),
        quotient_over_mk(kernel_quotient_relation(f.hom), a * b)) =
        quotient_over_mk(kernel_quotient_relation(f.hom), a.inverse * (a * b))
    a.inverse * (a * b) = a.inverse * a * b
    a.inverse * a = G.1
    a.inverse * a * b = G.1 * b
    G.1 * b = b
    quotient_over_mk(kernel_quotient_relation(f.hom), a.inverse * (a * b)) =
        quotient_over_mk(kernel_quotient_relation(f.hom), b)
}

/// Multiplication by a quotient inverse cancels on the right.
theorem group_hom_kernel_quotient_mul_inverse_cancel_right[G: Group, H: Group](
    f: GroupHom[G, H], a: G, b: G
) {
    group_hom_kernel_quotient_mul(f,
        group_hom_kernel_quotient_mul(f,
            quotient_over_mk(kernel_quotient_relation(f.hom), b),
            quotient_over_mk(kernel_quotient_relation(f.hom), a)),
        group_hom_kernel_quotient_inverse(f, quotient_over_mk(kernel_quotient_relation(f.hom), a))) =
        quotient_over_mk(kernel_quotient_relation(f.hom), b)
} by {
    group_hom_kernel_quotient_mul_projection(f, b, a)
    group_hom_kernel_quotient_mul(f,
        quotient_over_mk(kernel_quotient_relation(f.hom), b),
        quotient_over_mk(kernel_quotient_relation(f.hom), a)) =
        quotient_over_mk(kernel_quotient_relation(f.hom), b * a)
    group_hom_kernel_quotient_inverse_projection(f, a)
    group_hom_kernel_quotient_inverse(f, quotient_over_mk(kernel_quotient_relation(f.hom), a)) =
        quotient_over_mk(kernel_quotient_relation(f.hom), a.inverse)
    group_hom_kernel_quotient_mul(f,
        group_hom_kernel_quotient_mul(f,
            quotient_over_mk(kernel_quotient_relation(f.hom), b),
            quotient_over_mk(kernel_quotient_relation(f.hom), a)),
        group_hom_kernel_quotient_inverse(f, quotient_over_mk(kernel_quotient_relation(f.hom), a))) =
    group_hom_kernel_quotient_mul(f,
        quotient_over_mk(kernel_quotient_relation(f.hom), b * a),
        quotient_over_mk(kernel_quotient_relation(f.hom), a.inverse))
    group_hom_kernel_quotient_mul_projection(f, b * a, a.inverse)
    group_hom_kernel_quotient_mul(f,
        quotient_over_mk(kernel_quotient_relation(f.hom), b * a),
        quotient_over_mk(kernel_quotient_relation(f.hom), a.inverse)) =
        quotient_over_mk(kernel_quotient_relation(f.hom), (b * a) * a.inverse)
    (b * a) * a.inverse = b * (a * a.inverse)
    a * a.inverse = G.1
    b * (a * a.inverse) = b * G.1
    b * G.1 = b
    quotient_over_mk(kernel_quotient_relation(f.hom), (b * a) * a.inverse) =
        quotient_over_mk(kernel_quotient_relation(f.hom), b)
}

/// The kernel quotient relation of a semiring homomorphism respects addition.
theorem semiring_hom_kernel_quotient_relation_respects_add[S: Semiring, T: Semiring](
    f: SemiringHom[S, T]
) {
    quotient_binary_respects(kernel_quotient_relation(f.hom), S.add)
} by {
    quotient_binary_respects_eq_respects_binary_op(kernel_quotient_relation(f.hom), S.add)
    semiring_hom_kernel_respects_add(f)
    respects_add_eq_respects_binary_op(kernel_relation(f.hom))
    respects_binary_op(kernel_relation(f.hom), S.add)
    kernel_quotient_relation_rel(f.hom)
    kernel_quotient_relation(f.hom).rel = kernel_relation(f.hom)
    respects_binary_op(kernel_quotient_relation(f.hom).rel, S.add)
}

/// The kernel quotient relation of a semiring homomorphism respects multiplication.
theorem semiring_hom_kernel_quotient_relation_respects_mul[S: Semiring, T: Semiring](
    f: SemiringHom[S, T]
) {
    quotient_binary_respects(kernel_quotient_relation(f.hom), S.mul)
} by {
    quotient_binary_respects_eq_respects_binary_op(kernel_quotient_relation(f.hom), S.mul)
    semiring_hom_kernel_respects_mul(f)
    respects_mul_eq_respects_binary_op(kernel_relation(f.hom))
    respects_binary_op(kernel_relation(f.hom), S.mul)
    kernel_quotient_relation_rel(f.hom)
    kernel_quotient_relation(f.hom).rel = kernel_relation(f.hom)
    respects_binary_op(kernel_quotient_relation(f.hom).rel, S.mul)
}

/// Addition on the kernel quotient by a semiring homomorphism.
define semiring_hom_kernel_quotient_add[S: Semiring, T: Semiring](f: SemiringHom[S, T]) ->
    (QuotientOver[S], QuotientOver[S]) -> QuotientOver[S] {
    quotient_over_binary_op(S.add)
}

/// Multiplication on the kernel quotient by a semiring homomorphism.
define semiring_hom_kernel_quotient_mul[S: Semiring, T: Semiring](f: SemiringHom[S, T]) ->
    (QuotientOver[S], QuotientOver[S]) -> QuotientOver[S] {
    quotient_over_binary_op(S.mul)
}

/// The zero element on the kernel quotient by a semiring homomorphism.
define semiring_hom_kernel_quotient_zero[S: Semiring, T: Semiring](f: SemiringHom[S, T]) -> QuotientOver[S] {
    quotient_over_const(kernel_quotient_relation(f.hom), S.0)
}

/// The identity element on the kernel quotient by a semiring homomorphism.
define semiring_hom_kernel_quotient_one[S: Semiring, T: Semiring](f: SemiringHom[S, T]) -> QuotientOver[S] {
    quotient_over_const(kernel_quotient_relation(f.hom), S.1)
}

/// Powers on the kernel quotient by a semiring homomorphism.
define semiring_hom_kernel_quotient_pow[S: Semiring, T: Semiring](
    f: SemiringHom[S, T], q: QuotientOver[S], n: Nat
) -> QuotientOver[S] {
    match n {
        Nat.zero {
            semiring_hom_kernel_quotient_one(f)
        }
        Nat.suc(k) {
            semiring_hom_kernel_quotient_mul(f, q, semiring_hom_kernel_quotient_pow(f, q, k))
        }
    }
}

/// The zeroth power on the semiring-kernel quotient is the quotient identity.
theorem semiring_hom_kernel_quotient_pow_zero_projection[S: Semiring, T: Semiring](
    f: SemiringHom[S, T], q: QuotientOver[S]
) {
    semiring_hom_kernel_quotient_pow(f, q, Nat.0) = semiring_hom_kernel_quotient_one(f)
} by {
}

/// The successor power on the semiring-kernel quotient multiplies by the base.
theorem semiring_hom_kernel_quotient_pow_suc_projection[S: Semiring, T: Semiring](
    f: SemiringHom[S, T], q: QuotientOver[S], n: Nat
) {
    semiring_hom_kernel_quotient_pow(f, q, n.suc) =
        semiring_hom_kernel_quotient_mul(f, q, semiring_hom_kernel_quotient_pow(f, q, n))
}

/// The zeroth power on a projected semiring representative is the projected identity.
theorem semiring_hom_kernel_quotient_pow_mk_zero[S: Semiring, T: Semiring](f: SemiringHom[S, T], a: S) {
    semiring_hom_kernel_quotient_pow(f, quotient_over_mk(kernel_quotient_relation(f.hom), a), Nat.0) =
        quotient_over_mk(kernel_quotient_relation(f.hom), a.pow(Nat.0))
} by {
    semiring_hom_kernel_quotient_pow_zero_projection(f, quotient_over_mk(kernel_quotient_relation(f.hom), a))
    quotient_over_const_projection(kernel_quotient_relation(f.hom), S.1)
    a.pow(Nat.0) = S.1
}

/// Powers of projected semiring representatives agree with projected powers.
theorem semiring_hom_kernel_quotient_pow_mk[S: Semiring, T: Semiring](f: SemiringHom[S, T], a: S, n: Nat) {
    semiring_hom_kernel_quotient_pow(f, quotient_over_mk(kernel_quotient_relation(f.hom), a), n) =
        quotient_over_mk(kernel_quotient_relation(f.hom), a.pow(n))
} by {
    define p(k: Nat) -> Bool {
        semiring_hom_kernel_quotient_pow(f, quotient_over_mk(kernel_quotient_relation(f.hom), a), k) =
            quotient_over_mk(kernel_quotient_relation(f.hom), a.pow(k))
    }
    semiring_hom_kernel_quotient_pow_mk_zero(f, a)
    p(Nat.0)
    forall(k: Nat) {
        if p(k) {
            semiring_hom_kernel_quotient_pow_suc_projection(
                f, quotient_over_mk(kernel_quotient_relation(f.hom), a), k)
            semiring_hom_kernel_quotient_pow(f,
                quotient_over_mk(kernel_quotient_relation(f.hom), a), k.suc) =
                semiring_hom_kernel_quotient_mul(f,
                    quotient_over_mk(kernel_quotient_relation(f.hom), a),
                    semiring_hom_kernel_quotient_pow(f,
                        quotient_over_mk(kernel_quotient_relation(f.hom), a), k))
            semiring_hom_kernel_quotient_pow(f,
                quotient_over_mk(kernel_quotient_relation(f.hom), a), k) =
                quotient_over_mk(kernel_quotient_relation(f.hom), a.pow(k))
            semiring_hom_kernel_quotient_mul(f,
                quotient_over_mk(kernel_quotient_relation(f.hom), a),
                semiring_hom_kernel_quotient_pow(f,
                    quotient_over_mk(kernel_quotient_relation(f.hom), a), k)) =
                semiring_hom_kernel_quotient_mul(f,
                    quotient_over_mk(kernel_quotient_relation(f.hom), a),
                    quotient_over_mk(kernel_quotient_relation(f.hom), a.pow(k)))
            semiring_hom_kernel_quotient_relation_respects_mul(f)
            quotient_over_binary_op_projection(kernel_quotient_relation(f.hom), S.mul, a, a.pow(k))
            S.mul(a, a.pow(k)) = a * a.pow(k)
            semiring_hom_kernel_quotient_mul(f,
                quotient_over_mk(kernel_quotient_relation(f.hom), a),
                quotient_over_mk(kernel_quotient_relation(f.hom), a.pow(k))) =
                quotient_over_mk(kernel_quotient_relation(f.hom), a * a.pow(k))
            a.pow(k.suc) = a * a.pow(k)
            quotient_over_mk(kernel_quotient_relation(f.hom), a * a.pow(k)) =
                quotient_over_mk(kernel_quotient_relation(f.hom), a.pow(k.suc))
            p(k.suc)
        }
    }
    p(n)
}

/// Multiplying two projected semiring-kernel powers adds their exponents.
theorem semiring_hom_kernel_quotient_pow_add[S: Semiring, T: Semiring](
    f: SemiringHom[S, T], a: S, n: Nat, m: Nat
) {
    semiring_hom_kernel_quotient_mul(f,
        semiring_hom_kernel_quotient_pow(f, quotient_over_mk(kernel_quotient_relation(f.hom), a), n),
        semiring_hom_kernel_quotient_pow(f, quotient_over_mk(kernel_quotient_relation(f.hom), a), m)) =
    semiring_hom_kernel_quotient_pow(f, quotient_over_mk(kernel_quotient_relation(f.hom), a), n + m)
} by {
    semiring_hom_kernel_quotient_pow_mk(f, a, n)
    semiring_hom_kernel_quotient_pow_mk(f, a, m)
    semiring_hom_kernel_quotient_mul(f,
        semiring_hom_kernel_quotient_pow(f, quotient_over_mk(kernel_quotient_relation(f.hom), a), n),
        semiring_hom_kernel_quotient_pow(f, quotient_over_mk(kernel_quotient_relation(f.hom), a), m)) =
    semiring_hom_kernel_quotient_mul(f,
        quotient_over_mk(kernel_quotient_relation(f.hom), a.pow(n)),
        quotient_over_mk(kernel_quotient_relation(f.hom), a.pow(m)))
    semiring_hom_kernel_quotient_relation_respects_mul(f)
    quotient_over_binary_op_projection(kernel_quotient_relation(f.hom), S.mul, a.pow(n), a.pow(m))
    S.mul(a.pow(n), a.pow(m)) = a.pow(n) * a.pow(m)
    semiring_hom_kernel_quotient_mul(f,
        quotient_over_mk(kernel_quotient_relation(f.hom), a.pow(n)),
        quotient_over_mk(kernel_quotient_relation(f.hom), a.pow(m))) =
        quotient_over_mk(kernel_quotient_relation(f.hom), a.pow(n) * a.pow(m))
    semiring_hom_kernel_quotient_pow_mk(f, a, n + m)
    semiring_hom_kernel_quotient_pow(f, quotient_over_mk(kernel_quotient_relation(f.hom), a), n + m) =
        quotient_over_mk(kernel_quotient_relation(f.hom), a.pow(n + m))
    pow_add(a, n, m)
    a.pow(n) * a.pow(m) = a.pow(n + m)
    quotient_over_mk(kernel_quotient_relation(f.hom), a.pow(n) * a.pow(m)) =
        quotient_over_mk(kernel_quotient_relation(f.hom), a.pow(n + m))
}

/// Raising a projected semiring-kernel power to a power multiplies the exponents.
theorem semiring_hom_kernel_quotient_pow_pow[S: Semiring, T: Semiring](
    f: SemiringHom[S, T], a: S, n: Nat, m: Nat
) {
    semiring_hom_kernel_quotient_pow(f,
        semiring_hom_kernel_quotient_pow(f, quotient_over_mk(kernel_quotient_relation(f.hom), a), n),
        m) =
    semiring_hom_kernel_quotient_pow(f, quotient_over_mk(kernel_quotient_relation(f.hom), a), n * m)
} by {
    semiring_hom_kernel_quotient_pow_mk(f, a, n)
    semiring_hom_kernel_quotient_pow(f, quotient_over_mk(kernel_quotient_relation(f.hom), a), n) =
        quotient_over_mk(kernel_quotient_relation(f.hom), a.pow(n))
    semiring_hom_kernel_quotient_pow(f,
        semiring_hom_kernel_quotient_pow(f, quotient_over_mk(kernel_quotient_relation(f.hom), a), n),
        m) =
    semiring_hom_kernel_quotient_pow(f,
        quotient_over_mk(kernel_quotient_relation(f.hom), a.pow(n)),
        m)
    semiring_hom_kernel_quotient_pow_mk(f, a.pow(n), m)
    semiring_hom_kernel_quotient_pow(f,
        quotient_over_mk(kernel_quotient_relation(f.hom), a.pow(n)),
        m) =
        quotient_over_mk(kernel_quotient_relation(f.hom), a.pow(n).pow(m))
    semiring_hom_kernel_quotient_pow_mk(f, a, n * m)
    semiring_hom_kernel_quotient_pow(f, quotient_over_mk(kernel_quotient_relation(f.hom), a), n * m) =
        quotient_over_mk(kernel_quotient_relation(f.hom), a.pow(n * m))
    pow_pow(a, n, m)
    a.pow(n).pow(m) = a.pow(n * m)
    quotient_over_mk(kernel_quotient_relation(f.hom), a.pow(n).pow(m)) =
        quotient_over_mk(kernel_quotient_relation(f.hom), a.pow(n * m))
}

/// Quotient one raised to a natural power is quotient one in a semiring-kernel quotient.
theorem semiring_hom_kernel_quotient_one_pow[S: Semiring, T: Semiring](f: SemiringHom[S, T], n: Nat) {
    semiring_hom_kernel_quotient_pow(f, semiring_hom_kernel_quotient_one(f), n) =
        semiring_hom_kernel_quotient_one(f)
} by {
    quotient_over_const_projection(kernel_quotient_relation(f.hom), S.1)
    semiring_hom_kernel_quotient_pow(f, semiring_hom_kernel_quotient_one(f), n) =
    semiring_hom_kernel_quotient_pow(f, quotient_over_mk(kernel_quotient_relation(f.hom), S.1), n)
    semiring_hom_kernel_quotient_pow_mk(f, S.1, n)
    semiring_hom_kernel_quotient_pow(f, quotient_over_mk(kernel_quotient_relation(f.hom), S.1), n) =
        quotient_over_mk(kernel_quotient_relation(f.hom), S.1.pow(n))
    one_pow[S](n)
    S.1.pow(n) = S.1
    quotient_over_mk(kernel_quotient_relation(f.hom), S.1.pow(n)) =
        quotient_over_mk(kernel_quotient_relation(f.hom), S.1)
}

/// The zero element on the semiring-kernel quotient is the projection of zero.
theorem semiring_hom_kernel_quotient_zero_projection[S: Semiring, T: Semiring](f: SemiringHom[S, T]) {
    semiring_hom_kernel_quotient_zero(f) = quotient_over_mk(kernel_quotient_relation(f.hom), S.0)
} by {
    quotient_over_const_projection(kernel_quotient_relation(f.hom), S.0)
}

/// The identity element on the semiring-kernel quotient is the projection of one.
theorem semiring_hom_kernel_quotient_one_projection[S: Semiring, T: Semiring](f: SemiringHom[S, T]) {
    semiring_hom_kernel_quotient_one(f) = quotient_over_mk(kernel_quotient_relation(f.hom), S.1)
} by {
    quotient_over_const_projection(kernel_quotient_relation(f.hom), S.1)
}

/// Addition on the semiring-kernel quotient agrees with addition of representatives.
theorem semiring_hom_kernel_quotient_add_projection[S: Semiring, T: Semiring](
    f: SemiringHom[S, T], a: S, b: S
) {
    semiring_hom_kernel_quotient_add(f,
        quotient_over_mk(kernel_quotient_relation(f.hom), a),
        quotient_over_mk(kernel_quotient_relation(f.hom), b)
    ) = quotient_over_mk(kernel_quotient_relation(f.hom), a + b)
} by {
    semiring_hom_kernel_quotient_relation_respects_add(f)
    quotient_over_binary_op_projection(kernel_quotient_relation(f.hom), S.add, a, b)
    S.add(a, b) = a + b
}

/// Addition in the semiring-kernel quotient respects related left representatives.
theorem semiring_hom_kernel_quotient_add_projection_left[S: Semiring, T: Semiring](
    f: SemiringHom[S, T], a1: S, a2: S, b: S
) {
    kernel_relation(f.hom, a1, a2) implies
    semiring_hom_kernel_quotient_add(f,
        quotient_over_mk(kernel_quotient_relation(f.hom), a1),
        quotient_over_mk(kernel_quotient_relation(f.hom), b)
    ) = semiring_hom_kernel_quotient_add(f,
        quotient_over_mk(kernel_quotient_relation(f.hom), a2),
        quotient_over_mk(kernel_quotient_relation(f.hom), b)
    )
} by {
    if kernel_relation(f.hom, a1, a2) {
        kernel_quotient_relation_rel(f.hom)
        kernel_quotient_relation(f.hom).rel(a1, a2)
        semiring_hom_kernel_quotient_relation_respects_add(f)
        quotient_over_binary_op_projection_left(kernel_quotient_relation(f.hom), S.add, a1, a2, b)
    }
}

/// Addition in the semiring-kernel quotient respects related right representatives.
theorem semiring_hom_kernel_quotient_add_projection_right[S: Semiring, T: Semiring](
    f: SemiringHom[S, T], a: S, b1: S, b2: S
) {
    kernel_relation(f.hom, b1, b2) implies
    semiring_hom_kernel_quotient_add(f,
        quotient_over_mk(kernel_quotient_relation(f.hom), a),
        quotient_over_mk(kernel_quotient_relation(f.hom), b1)
    ) = semiring_hom_kernel_quotient_add(f,
        quotient_over_mk(kernel_quotient_relation(f.hom), a),
        quotient_over_mk(kernel_quotient_relation(f.hom), b2)
    )
} by {
    if kernel_relation(f.hom, b1, b2) {
        kernel_quotient_relation_rel(f.hom)
        kernel_quotient_relation(f.hom).rel(b1, b2)
        semiring_hom_kernel_quotient_relation_respects_add(f)
        quotient_over_binary_op_projection_right(kernel_quotient_relation(f.hom), S.add, a, b1, b2)
    }
}

/// Multiplication on the semiring-kernel quotient agrees with multiplication of representatives.
theorem semiring_hom_kernel_quotient_mul_projection[S: Semiring, T: Semiring](
    f: SemiringHom[S, T], a: S, b: S
) {
    semiring_hom_kernel_quotient_mul(f,
        quotient_over_mk(kernel_quotient_relation(f.hom), a),
        quotient_over_mk(kernel_quotient_relation(f.hom), b)
    ) = quotient_over_mk(kernel_quotient_relation(f.hom), a * b)
} by {
    semiring_hom_kernel_quotient_relation_respects_mul(f)
    quotient_over_binary_op_projection(kernel_quotient_relation(f.hom), S.mul, a, b)
    S.mul(a, b) = a * b
}

/// Multiplication in the semiring-kernel quotient respects related left representatives.
theorem semiring_hom_kernel_quotient_mul_projection_left[S: Semiring, T: Semiring](
    f: SemiringHom[S, T], a1: S, a2: S, b: S
) {
    kernel_relation(f.hom, a1, a2) implies
    semiring_hom_kernel_quotient_mul(f,
        quotient_over_mk(kernel_quotient_relation(f.hom), a1),
        quotient_over_mk(kernel_quotient_relation(f.hom), b)
    ) = semiring_hom_kernel_quotient_mul(f,
        quotient_over_mk(kernel_quotient_relation(f.hom), a2),
        quotient_over_mk(kernel_quotient_relation(f.hom), b)
    )
} by {
    if kernel_relation(f.hom, a1, a2) {
        kernel_quotient_relation_rel(f.hom)
        kernel_quotient_relation(f.hom).rel(a1, a2)
        semiring_hom_kernel_quotient_relation_respects_mul(f)
        quotient_over_binary_op_projection_left(kernel_quotient_relation(f.hom), S.mul, a1, a2, b)
    }
}

/// Multiplication in the semiring-kernel quotient respects related right representatives.
theorem semiring_hom_kernel_quotient_mul_projection_right[S: Semiring, T: Semiring](
    f: SemiringHom[S, T], a: S, b1: S, b2: S
) {
    kernel_relation(f.hom, b1, b2) implies
    semiring_hom_kernel_quotient_mul(f,
        quotient_over_mk(kernel_quotient_relation(f.hom), a),
        quotient_over_mk(kernel_quotient_relation(f.hom), b1)
    ) = semiring_hom_kernel_quotient_mul(f,
        quotient_over_mk(kernel_quotient_relation(f.hom), a),
        quotient_over_mk(kernel_quotient_relation(f.hom), b2)
    )
} by {
    if kernel_relation(f.hom, b1, b2) {
        kernel_quotient_relation_rel(f.hom)
        kernel_quotient_relation(f.hom).rel(b1, b2)
        semiring_hom_kernel_quotient_relation_respects_mul(f)
        quotient_over_binary_op_projection_right(kernel_quotient_relation(f.hom), S.mul, a, b1, b2)
    }
}

/// The canonical semiring-kernel projection.
define semiring_hom_kernel_quotient_project[S: Semiring, T: Semiring](
    f: SemiringHom[S, T], a: S
) -> QuotientOver[S] {
    quotient_over_mk(kernel_quotient_relation(f.hom), a)
}

/// The canonical semiring-kernel projection is the quotient constructor.
theorem semiring_hom_kernel_quotient_project_eq_mk[S: Semiring, T: Semiring](
    f: SemiringHom[S, T], a: S
) {
    semiring_hom_kernel_quotient_project(f, a) =
        quotient_over_mk(kernel_quotient_relation(f.hom), a)
}

/// Equality of projected semiring-kernel representatives is the kernel relation.
theorem semiring_hom_kernel_quotient_project_eq_iff_kernel_relation[S: Semiring, T: Semiring](
    f: SemiringHom[S, T], a: S, b: S
) {
    (semiring_hom_kernel_quotient_project(f, a) = semiring_hom_kernel_quotient_project(f, b)) =
        kernel_relation(f.hom, a, b)
} by {
    semiring_hom_kernel_quotient_project_eq_mk(f, a)
    semiring_hom_kernel_quotient_project_eq_mk(f, b)
    quotient_over_mk_eq_iff_rel(kernel_quotient_relation(f.hom), a, b)
    kernel_quotient_relation_rel(f.hom)
}

/// The homomorphism out of the semiring-kernel quotient induced by `f`.
define semiring_hom_kernel_quotient_induced[S: Semiring, T: Semiring](f: SemiringHom[S, T]) ->
    QuotientOver[S] -> T {
    quotient_over_lift_to(f.hom)
}

/// The induced map agrees with `f` on canonical semiring-kernel projections.
theorem semiring_hom_kernel_quotient_induced_project[S: Semiring, T: Semiring](
    f: SemiringHom[S, T], a: S
) {
    semiring_hom_kernel_quotient_induced(f, semiring_hom_kernel_quotient_project(f, a)) = f.hom(a)
} by {
    kernel_relation_respects(f.hom)
    semiring_hom_kernel_quotient_project_eq_mk(f, a)
    quotient_over_lift_to_projection(kernel_quotient_relation(f.hom), f.hom, a)
}

/// The induced map out of the semiring-kernel quotient separates projections of unrelated representatives.
theorem semiring_hom_kernel_quotient_induced_injective[S: Semiring, T: Semiring](
    f: SemiringHom[S, T], a: S, b: S
) {
    semiring_hom_kernel_quotient_induced(f, semiring_hom_kernel_quotient_project(f, a)) =
        semiring_hom_kernel_quotient_induced(f, semiring_hom_kernel_quotient_project(f, b))
    implies semiring_hom_kernel_quotient_project(f, a) = semiring_hom_kernel_quotient_project(f, b)
} by {
    if semiring_hom_kernel_quotient_induced(f, semiring_hom_kernel_quotient_project(f, a)) =
        semiring_hom_kernel_quotient_induced(f, semiring_hom_kernel_quotient_project(f, b)) {
        semiring_hom_kernel_quotient_induced_project(f, a)
        semiring_hom_kernel_quotient_induced_project(f, b)
        f.hom(a) = f.hom(b)
        kernel_relation(f.hom, a, b) = (f.hom(a) = f.hom(b))
        kernel_relation(f.hom, a, b)
        semiring_hom_kernel_quotient_project_eq_iff_kernel_relation(f, a, b)
        semiring_hom_kernel_quotient_project(f, a) = semiring_hom_kernel_quotient_project(f, b)
    }
}

/// The canonical semiring-kernel projection preserves zero.
theorem semiring_hom_kernel_quotient_project_zero[S: Semiring, T: Semiring](f: SemiringHom[S, T]) {
    semiring_hom_kernel_quotient_project(f, S.0) = semiring_hom_kernel_quotient_zero(f)
} by {
    semiring_hom_kernel_quotient_zero_projection(f)
}

/// The canonical semiring-kernel projection preserves one.
theorem semiring_hom_kernel_quotient_project_one[S: Semiring, T: Semiring](f: SemiringHom[S, T]) {
    semiring_hom_kernel_quotient_project(f, S.1) = semiring_hom_kernel_quotient_one(f)
} by {
    semiring_hom_kernel_quotient_one_projection(f)
}

/// The canonical semiring-kernel projection preserves addition.
theorem semiring_hom_kernel_quotient_project_add[S: Semiring, T: Semiring](
    f: SemiringHom[S, T], a: S, b: S
) {
    semiring_hom_kernel_quotient_project(f, a + b) =
    semiring_hom_kernel_quotient_add(f,
        semiring_hom_kernel_quotient_project(f, a),
        semiring_hom_kernel_quotient_project(f, b))
} by {
    semiring_hom_kernel_quotient_add_projection(f, a, b)
    semiring_hom_kernel_quotient_project(f, a + b) =
        quotient_over_mk(kernel_quotient_relation(f.hom), a + b)
    semiring_hom_kernel_quotient_project(f, a) =
        quotient_over_mk(kernel_quotient_relation(f.hom), a)
    semiring_hom_kernel_quotient_project(f, b) =
        quotient_over_mk(kernel_quotient_relation(f.hom), b)
}

/// The canonical semiring-kernel projection preserves multiplication.
theorem semiring_hom_kernel_quotient_project_mul[S: Semiring, T: Semiring](
    f: SemiringHom[S, T], a: S, b: S
) {
    semiring_hom_kernel_quotient_project(f, a * b) =
    semiring_hom_kernel_quotient_mul(f,
        semiring_hom_kernel_quotient_project(f, a),
        semiring_hom_kernel_quotient_project(f, b))
} by {
    semiring_hom_kernel_quotient_mul_projection(f, a, b)
    semiring_hom_kernel_quotient_project(f, a * b) =
        quotient_over_mk(kernel_quotient_relation(f.hom), a * b)
    semiring_hom_kernel_quotient_project(f, a) =
        quotient_over_mk(kernel_quotient_relation(f.hom), a)
    semiring_hom_kernel_quotient_project(f, b) =
        quotient_over_mk(kernel_quotient_relation(f.hom), b)
}

/// The induced map out of the semiring-kernel quotient preserves addition.
theorem semiring_hom_kernel_quotient_induced_add[S: Semiring, T: Semiring](
    f: SemiringHom[S, T], a: S, b: S
) {
    semiring_hom_kernel_quotient_induced(f,
        semiring_hom_kernel_quotient_add(f,
            semiring_hom_kernel_quotient_project(f, a),
            semiring_hom_kernel_quotient_project(f, b))) =
        semiring_hom_kernel_quotient_induced(f, semiring_hom_kernel_quotient_project(f, a)) +
        semiring_hom_kernel_quotient_induced(f, semiring_hom_kernel_quotient_project(f, b))
} by {
    semiring_hom_kernel_quotient_project_add(f, a, b)
    semiring_hom_kernel_quotient_induced_project(f, a + b)
    semiring_hom_kernel_quotient_induced_project(f, a)
    semiring_hom_kernel_quotient_induced_project(f, b)
    semiring_hom_add(f, a, b)
}

/// The induced map out of the semiring-kernel quotient preserves multiplication.
theorem semiring_hom_kernel_quotient_induced_mul[S: Semiring, T: Semiring](
    f: SemiringHom[S, T], a: S, b: S
) {
    semiring_hom_kernel_quotient_induced(f,
        semiring_hom_kernel_quotient_mul(f,
            semiring_hom_kernel_quotient_project(f, a),
            semiring_hom_kernel_quotient_project(f, b))) =
        semiring_hom_kernel_quotient_induced(f, semiring_hom_kernel_quotient_project(f, a)) *
        semiring_hom_kernel_quotient_induced(f, semiring_hom_kernel_quotient_project(f, b))
} by {
    semiring_hom_kernel_quotient_project_mul(f, a, b)
    semiring_hom_kernel_quotient_induced_project(f, a * b)
    semiring_hom_kernel_quotient_induced_project(f, a)
    semiring_hom_kernel_quotient_induced_project(f, b)
    semiring_hom_mul(f, a, b)
}

/// The induced map out of the semiring-kernel quotient preserves zero.
theorem semiring_hom_kernel_quotient_induced_zero[S: Semiring, T: Semiring](f: SemiringHom[S, T]) {
    semiring_hom_kernel_quotient_induced(f, semiring_hom_kernel_quotient_zero(f)) = T.0
} by {
    semiring_hom_kernel_quotient_project_zero(f)
    semiring_hom_kernel_quotient_induced_project(f, S.0)
    semiring_hom_zero(f)
}

/// The induced map out of the semiring-kernel quotient preserves one.
theorem semiring_hom_kernel_quotient_induced_one[S: Semiring, T: Semiring](f: SemiringHom[S, T]) {
    semiring_hom_kernel_quotient_induced(f, semiring_hom_kernel_quotient_one(f)) = T.1
} by {
    semiring_hom_kernel_quotient_project_one(f)
    semiring_hom_kernel_quotient_induced_project(f, S.1)
    semiring_hom_one(f)
}

/// Every value of `f` is attained by the induced map out of the semiring-kernel quotient.
theorem semiring_hom_kernel_quotient_induced_surjective_onto_image[S: Semiring, T: Semiring](
    f: SemiringHom[S, T], a: S
) {
    exists(q: QuotientOver[S]) {
        semiring_hom_kernel_quotient_induced(f, q) = f.hom(a)
    }
} by {
    semiring_hom_kernel_quotient_induced_project(f, a)
    semiring_hom_kernel_quotient_induced(f, semiring_hom_kernel_quotient_project(f, a)) = f.hom(a)
}

/// Addition of canonical semiring-kernel projections respects related left representatives.
theorem semiring_hom_kernel_quotient_project_add_left_of_kernel_relation[S: Semiring, T: Semiring](
    f: SemiringHom[S, T], a1: S, a2: S, b: S
) {
    kernel_relation(f.hom, a1, a2) implies
    semiring_hom_kernel_quotient_add(f,
        semiring_hom_kernel_quotient_project(f, a1),
        semiring_hom_kernel_quotient_project(f, b)) =
    semiring_hom_kernel_quotient_add(f,
        semiring_hom_kernel_quotient_project(f, a2),
        semiring_hom_kernel_quotient_project(f, b))
} by {
    if kernel_relation(f.hom, a1, a2) {
        semiring_hom_kernel_quotient_add_projection_left(f, a1, a2, b)
    }
}

/// Addition of canonical semiring-kernel projections respects related right representatives.
theorem semiring_hom_kernel_quotient_project_add_right_of_kernel_relation[S: Semiring, T: Semiring](
    f: SemiringHom[S, T], a: S, b1: S, b2: S
) {
    kernel_relation(f.hom, b1, b2) implies
    semiring_hom_kernel_quotient_add(f,
        semiring_hom_kernel_quotient_project(f, a),
        semiring_hom_kernel_quotient_project(f, b1)) =
    semiring_hom_kernel_quotient_add(f,
        semiring_hom_kernel_quotient_project(f, a),
        semiring_hom_kernel_quotient_project(f, b2))
} by {
    if kernel_relation(f.hom, b1, b2) {
        semiring_hom_kernel_quotient_add_projection_right(f, a, b1, b2)
    }
}

/// Multiplication of canonical semiring-kernel projections respects related left representatives.
theorem semiring_hom_kernel_quotient_project_mul_left_of_kernel_relation[S: Semiring, T: Semiring](
    f: SemiringHom[S, T], a1: S, a2: S, b: S
) {
    kernel_relation(f.hom, a1, a2) implies
    semiring_hom_kernel_quotient_mul(f,
        semiring_hom_kernel_quotient_project(f, a1),
        semiring_hom_kernel_quotient_project(f, b)) =
    semiring_hom_kernel_quotient_mul(f,
        semiring_hom_kernel_quotient_project(f, a2),
        semiring_hom_kernel_quotient_project(f, b))
} by {
    if kernel_relation(f.hom, a1, a2) {
        semiring_hom_kernel_quotient_mul_projection_left(f, a1, a2, b)
    }
}

/// Multiplication of canonical semiring-kernel projections respects related right representatives.
theorem semiring_hom_kernel_quotient_project_mul_right_of_kernel_relation[S: Semiring, T: Semiring](
    f: SemiringHom[S, T], a: S, b1: S, b2: S
) {
    kernel_relation(f.hom, b1, b2) implies
    semiring_hom_kernel_quotient_mul(f,
        semiring_hom_kernel_quotient_project(f, a),
        semiring_hom_kernel_quotient_project(f, b1)) =
    semiring_hom_kernel_quotient_mul(f,
        semiring_hom_kernel_quotient_project(f, a),
        semiring_hom_kernel_quotient_project(f, b2))
} by {
    if kernel_relation(f.hom, b1, b2) {
        semiring_hom_kernel_quotient_mul_projection_right(f, a, b1, b2)
    }
}

/// The zeroth power of a projected semiring-kernel representative is the projected identity.
theorem semiring_hom_kernel_quotient_project_pow_zero[S: Semiring, T: Semiring](
    f: SemiringHom[S, T], a: S
) {
    semiring_hom_kernel_quotient_pow(f, semiring_hom_kernel_quotient_project(f, a), Nat.0) =
        semiring_hom_kernel_quotient_project(f, a.pow(Nat.0))
} by {
    semiring_hom_kernel_quotient_pow_mk_zero(f, a)
}

/// Powers of projected semiring-kernel representatives agree with projected powers.
theorem semiring_hom_kernel_quotient_project_pow[S: Semiring, T: Semiring](
    f: SemiringHom[S, T], a: S, n: Nat
) {
    semiring_hom_kernel_quotient_pow(f, semiring_hom_kernel_quotient_project(f, a), n) =
        semiring_hom_kernel_quotient_project(f, a.pow(n))
} by {
    semiring_hom_kernel_quotient_pow_mk(f, a, n)
}

/// The first power of a projected semiring-kernel representative is the projected representative.
theorem semiring_hom_kernel_quotient_project_pow_one[S: Semiring, T: Semiring](
    f: SemiringHom[S, T], a: S
) {
    semiring_hom_kernel_quotient_pow(f, semiring_hom_kernel_quotient_project(f, a), Nat.1) =
        semiring_hom_kernel_quotient_project(f, a)
} by {
    semiring_hom_kernel_quotient_project_pow(f, a, Nat.1)
    a.pow(Nat.1) = a
}

/// Quotient one raised to a natural power is quotient one for the canonical projection.
theorem semiring_hom_kernel_quotient_project_one_pow[S: Semiring, T: Semiring](
    f: SemiringHom[S, T], n: Nat
) {
    semiring_hom_kernel_quotient_pow(f, semiring_hom_kernel_quotient_one(f), n) =
        semiring_hom_kernel_quotient_one(f)
} by {
    semiring_hom_kernel_quotient_one_pow(f, n)
}

/// Multiplying projected semiring-kernel quotient powers adds their exponents.
theorem semiring_hom_kernel_quotient_project_pow_add[S: Semiring, T: Semiring](
    f: SemiringHom[S, T], a: S, n: Nat, m: Nat
) {
    semiring_hom_kernel_quotient_mul(f,
        semiring_hom_kernel_quotient_pow(f, semiring_hom_kernel_quotient_project(f, a), n),
        semiring_hom_kernel_quotient_pow(f, semiring_hom_kernel_quotient_project(f, a), m)) =
    semiring_hom_kernel_quotient_pow(f, semiring_hom_kernel_quotient_project(f, a), n + m)
} by {
    semiring_hom_kernel_quotient_pow_add(f, a, n, m)
}

/// Raising a projected semiring-kernel quotient power to a power multiplies the exponents.
theorem semiring_hom_kernel_quotient_project_pow_pow[S: Semiring, T: Semiring](
    f: SemiringHom[S, T], a: S, n: Nat, m: Nat
) {
    semiring_hom_kernel_quotient_pow(f,
        semiring_hom_kernel_quotient_pow(f, semiring_hom_kernel_quotient_project(f, a), n),
        m) =
    semiring_hom_kernel_quotient_pow(f, semiring_hom_kernel_quotient_project(f, a), n * m)
} by {
    semiring_hom_kernel_quotient_pow_pow(f, a, n, m)
}

/// Adding quotient zero on the left fixes a projected semiring-kernel representative.
theorem semiring_hom_kernel_quotient_project_zero_add[S: Semiring, T: Semiring](
    f: SemiringHom[S, T], a: S
) {
    semiring_hom_kernel_quotient_add(f,
        semiring_hom_kernel_quotient_zero(f),
        semiring_hom_kernel_quotient_project(f, a)) =
        semiring_hom_kernel_quotient_project(f, a)
} by {
    semiring_hom_kernel_quotient_add_projection(f, S.0, a)
    semiring_hom_kernel_quotient_zero_projection(f)
    S.0 + a = a
}

/// Adding quotient zero on the right fixes a projected semiring-kernel representative.
theorem semiring_hom_kernel_quotient_project_add_zero[S: Semiring, T: Semiring](
    f: SemiringHom[S, T], a: S
) {
    semiring_hom_kernel_quotient_add(f,
        semiring_hom_kernel_quotient_project(f, a),
        semiring_hom_kernel_quotient_zero(f)) =
        semiring_hom_kernel_quotient_project(f, a)
} by {
    semiring_hom_kernel_quotient_add_projection(f, a, S.0)
    semiring_hom_kernel_quotient_zero_projection(f)
    a + S.0 = a
}

/// Multiplying by quotient one on the left fixes a projected semiring-kernel representative.
theorem semiring_hom_kernel_quotient_project_one_mul[S: Semiring, T: Semiring](
    f: SemiringHom[S, T], a: S
) {
    semiring_hom_kernel_quotient_mul(f,
        semiring_hom_kernel_quotient_one(f),
        semiring_hom_kernel_quotient_project(f, a)) =
        semiring_hom_kernel_quotient_project(f, a)
} by {
    semiring_hom_kernel_quotient_mul_projection(f, S.1, a)
    semiring_hom_kernel_quotient_one_projection(f)
    S.1 * a = a
}

/// Multiplying by quotient one on the right fixes a projected semiring-kernel representative.
theorem semiring_hom_kernel_quotient_project_mul_one[S: Semiring, T: Semiring](
    f: SemiringHom[S, T], a: S
) {
    semiring_hom_kernel_quotient_mul(f,
        semiring_hom_kernel_quotient_project(f, a),
        semiring_hom_kernel_quotient_one(f)) =
        semiring_hom_kernel_quotient_project(f, a)
} by {
    semiring_hom_kernel_quotient_mul_projection(f, a, S.1)
    semiring_hom_kernel_quotient_one_projection(f)
    a * S.1 = a
}

/// Multiplying by quotient zero on the left gives quotient zero.
theorem semiring_hom_kernel_quotient_project_zero_mul[S: Semiring, T: Semiring](
    f: SemiringHom[S, T], a: S
) {
    semiring_hom_kernel_quotient_mul(f,
        semiring_hom_kernel_quotient_zero(f),
        semiring_hom_kernel_quotient_project(f, a)) =
        semiring_hom_kernel_quotient_zero(f)
} by {
    semiring_hom_kernel_quotient_mul_projection(f, S.0, a)
    semiring_hom_kernel_quotient_zero_projection(f)
    S.0 * a = S.0
}

/// Multiplying by quotient zero on the right gives quotient zero.
theorem semiring_hom_kernel_quotient_project_mul_zero[S: Semiring, T: Semiring](
    f: SemiringHom[S, T], a: S
) {
    semiring_hom_kernel_quotient_mul(f,
        semiring_hom_kernel_quotient_project(f, a),
        semiring_hom_kernel_quotient_zero(f)) =
        semiring_hom_kernel_quotient_zero(f)
} by {
    semiring_hom_kernel_quotient_mul_projection(f, a, S.0)
    semiring_hom_kernel_quotient_zero_projection(f)
    a * S.0 = S.0
}

/// Addition of projected semiring-kernel representatives is associative.
theorem semiring_hom_kernel_quotient_project_add_assoc[S: Semiring, T: Semiring](
    f: SemiringHom[S, T], a: S, b: S, c: S
) {
    semiring_hom_kernel_quotient_add(f,
        semiring_hom_kernel_quotient_add(f,
            semiring_hom_kernel_quotient_project(f, a),
            semiring_hom_kernel_quotient_project(f, b)),
        semiring_hom_kernel_quotient_project(f, c)) =
    semiring_hom_kernel_quotient_add(f,
        semiring_hom_kernel_quotient_project(f, a),
        semiring_hom_kernel_quotient_add(f,
            semiring_hom_kernel_quotient_project(f, b),
            semiring_hom_kernel_quotient_project(f, c)))
} by {
    semiring_hom_kernel_quotient_add_projection(f, a, b)
    semiring_hom_kernel_quotient_add_projection(f, a + b, c)
    semiring_hom_kernel_quotient_add_projection(f, b, c)
    semiring_hom_kernel_quotient_add_projection(f, a, b + c)
    (a + b) + c = a + (b + c)
}

/// Multiplication of projected semiring-kernel representatives is associative.
theorem semiring_hom_kernel_quotient_project_mul_assoc[S: Semiring, T: Semiring](
    f: SemiringHom[S, T], a: S, b: S, c: S
) {
    semiring_hom_kernel_quotient_mul(f,
        semiring_hom_kernel_quotient_mul(f,
            semiring_hom_kernel_quotient_project(f, a),
            semiring_hom_kernel_quotient_project(f, b)),
        semiring_hom_kernel_quotient_project(f, c)) =
    semiring_hom_kernel_quotient_mul(f,
        semiring_hom_kernel_quotient_project(f, a),
        semiring_hom_kernel_quotient_mul(f,
            semiring_hom_kernel_quotient_project(f, b),
            semiring_hom_kernel_quotient_project(f, c)))
} by {
    semiring_hom_kernel_quotient_mul_projection(f, a, b)
    semiring_hom_kernel_quotient_mul_projection(f, a * b, c)
    semiring_hom_kernel_quotient_mul_projection(f, b, c)
    semiring_hom_kernel_quotient_mul_projection(f, a, b * c)
    (a * b) * c = a * (b * c)
}

/// Addition of projected semiring-kernel representatives is commutative.
theorem semiring_hom_kernel_quotient_project_add_comm[S: Semiring, T: Semiring](
    f: SemiringHom[S, T], a: S, b: S
) {
    semiring_hom_kernel_quotient_add(f,
        semiring_hom_kernel_quotient_project(f, a),
        semiring_hom_kernel_quotient_project(f, b)) =
    semiring_hom_kernel_quotient_add(f,
        semiring_hom_kernel_quotient_project(f, b),
        semiring_hom_kernel_quotient_project(f, a))
} by {
    semiring_hom_kernel_quotient_add_projection(f, a, b)
    semiring_hom_kernel_quotient_add_projection(f, b, a)
    a + b = b + a
}

/// Multiplication distributes over quotient addition from the left.
theorem semiring_hom_kernel_quotient_project_distrib_left[S: Semiring, T: Semiring](
    f: SemiringHom[S, T], a: S, b: S, c: S
) {
    semiring_hom_kernel_quotient_mul(f,
        semiring_hom_kernel_quotient_project(f, a),
        semiring_hom_kernel_quotient_add(f,
            semiring_hom_kernel_quotient_project(f, b),
            semiring_hom_kernel_quotient_project(f, c))) =
    semiring_hom_kernel_quotient_add(f,
        semiring_hom_kernel_quotient_mul(f,
            semiring_hom_kernel_quotient_project(f, a),
            semiring_hom_kernel_quotient_project(f, b)),
        semiring_hom_kernel_quotient_mul(f,
            semiring_hom_kernel_quotient_project(f, a),
            semiring_hom_kernel_quotient_project(f, c)))
} by {
    semiring_hom_kernel_quotient_add_projection(f, b, c)
    semiring_hom_kernel_quotient_mul_projection(f, a, b + c)
    semiring_hom_kernel_quotient_mul_projection(f, a, b)
    semiring_hom_kernel_quotient_mul_projection(f, a, c)
    semiring_hom_kernel_quotient_add_projection(f, a * b, a * c)
    a * (b + c) = (a * b) + (a * c)
}

/// Multiplication distributes over quotient addition from the right.
theorem semiring_hom_kernel_quotient_project_distrib_right[S: Semiring, T: Semiring](
    f: SemiringHom[S, T], a: S, b: S, c: S
) {
    semiring_hom_kernel_quotient_mul(f,
        semiring_hom_kernel_quotient_add(f,
            semiring_hom_kernel_quotient_project(f, a),
            semiring_hom_kernel_quotient_project(f, b)),
        semiring_hom_kernel_quotient_project(f, c)) =
    semiring_hom_kernel_quotient_add(f,
        semiring_hom_kernel_quotient_mul(f,
            semiring_hom_kernel_quotient_project(f, a),
            semiring_hom_kernel_quotient_project(f, c)),
        semiring_hom_kernel_quotient_mul(f,
            semiring_hom_kernel_quotient_project(f, b),
            semiring_hom_kernel_quotient_project(f, c)))
} by {
    semiring_hom_kernel_quotient_add_projection(f, a, b)
    semiring_hom_kernel_quotient_mul_projection(f, a + b, c)
    semiring_hom_kernel_quotient_mul_projection(f, a, c)
    semiring_hom_kernel_quotient_mul_projection(f, b, c)
    semiring_hom_kernel_quotient_add_projection(f, a * c, b * c)
    (a + b) * c = (a * c) + (b * c)
}

/// The kernel quotient relation of a ring homomorphism respects addition.
theorem ring_hom_kernel_quotient_relation_respects_add[R: Ring, S: Ring](f: RingHom[R, S]) {
    quotient_binary_respects(kernel_quotient_relation(f.hom), R.add)
} by {
    quotient_binary_respects_eq_respects_binary_op(kernel_quotient_relation(f.hom), R.add)
    ring_hom_kernel_respects_add(f)
    respects_add_eq_respects_binary_op(kernel_relation(f.hom))
    respects_binary_op(kernel_relation(f.hom), R.add)
    kernel_quotient_relation_rel(f.hom)
    kernel_quotient_relation(f.hom).rel = kernel_relation(f.hom)
    respects_binary_op(kernel_quotient_relation(f.hom).rel, R.add)
}

/// The kernel quotient relation of a ring homomorphism respects multiplication.
theorem ring_hom_kernel_quotient_relation_respects_mul[R: Ring, S: Ring](f: RingHom[R, S]) {
    quotient_binary_respects(kernel_quotient_relation(f.hom), R.mul)
} by {
    quotient_binary_respects_eq_respects_binary_op(kernel_quotient_relation(f.hom), R.mul)
    ring_hom_kernel_respects_mul(f)
    respects_mul_eq_respects_binary_op(kernel_relation(f.hom))
    respects_binary_op(kernel_relation(f.hom), R.mul)
    kernel_quotient_relation_rel(f.hom)
    kernel_quotient_relation(f.hom).rel = kernel_relation(f.hom)
    respects_binary_op(kernel_quotient_relation(f.hom).rel, R.mul)
}

/// The kernel quotient relation of a ring homomorphism respects negation.
theorem ring_hom_kernel_quotient_relation_respects_neg[R: Ring, S: Ring](f: RingHom[R, S]) {
    quotient_unary_respects(kernel_quotient_relation(f.hom), R.neg)
} by {
    quotient_unary_respects_eq_respects_unary_op(kernel_quotient_relation(f.hom), R.neg)
    let fa: AddGroupHom[R, S] = ring_hom_to_add_group_hom(f)
    ring_hom_to_add_group_hom_hom(f)
    fa.hom = f.hom
    add_group_hom_kernel_respects_neg(fa)
    respects_unary_op(kernel_relation(fa.hom), R.neg)
    kernel_relation(fa.hom) = kernel_relation(f.hom)
    respects_unary_op(kernel_relation(f.hom), R.neg)
    kernel_quotient_relation_rel(f.hom)
    kernel_quotient_relation(f.hom).rel = kernel_relation(f.hom)
    respects_unary_op(kernel_quotient_relation(f.hom).rel, R.neg)
}

/// Addition on the kernel quotient by a ring homomorphism.
define ring_hom_kernel_quotient_add[R: Ring, S: Ring](f: RingHom[R, S]) ->
    (QuotientOver[R], QuotientOver[R]) -> QuotientOver[R] {
    quotient_over_binary_op(R.add)
}

/// The zero element on the kernel quotient by a ring homomorphism.
define ring_hom_kernel_quotient_zero[R: Ring, S: Ring](f: RingHom[R, S]) -> QuotientOver[R] {
    quotient_over_const(kernel_quotient_relation(f.hom), R.0)
}

/// The zero element on the kernel quotient is the projection of zero.
theorem ring_hom_kernel_quotient_zero_projection[R: Ring, S: Ring](f: RingHom[R, S]) {
    ring_hom_kernel_quotient_zero(f) = quotient_over_mk(kernel_quotient_relation(f.hom), R.0)
} by {
    quotient_over_const_projection(kernel_quotient_relation(f.hom), R.0)
}

/// Addition on the kernel quotient agrees with addition of representatives.
theorem ring_hom_kernel_quotient_add_projection[R: Ring, S: Ring](f: RingHom[R, S], a: R, b: R) {
    ring_hom_kernel_quotient_add(f,
        quotient_over_mk(kernel_quotient_relation(f.hom), a),
        quotient_over_mk(kernel_quotient_relation(f.hom), b)
    ) = quotient_over_mk(kernel_quotient_relation(f.hom), a + b)
} by {
    ring_hom_kernel_quotient_relation_respects_add(f)
    quotient_over_binary_op_projection(kernel_quotient_relation(f.hom), R.add, a, b)
    R.add(a, b) = a + b
}

/// Addition in the kernel quotient respects related left representatives.
theorem ring_hom_kernel_quotient_add_projection_left[R: Ring, S: Ring](f: RingHom[R, S],
        a1: R, a2: R, b: R) {
    kernel_relation(f.hom, a1, a2) implies
    ring_hom_kernel_quotient_add(f,
        quotient_over_mk(kernel_quotient_relation(f.hom), a1),
        quotient_over_mk(kernel_quotient_relation(f.hom), b)
    ) = ring_hom_kernel_quotient_add(f,
        quotient_over_mk(kernel_quotient_relation(f.hom), a2),
        quotient_over_mk(kernel_quotient_relation(f.hom), b)
    )
} by {
    if kernel_relation(f.hom, a1, a2) {
        kernel_quotient_relation_rel(f.hom)
        kernel_quotient_relation(f.hom).rel(a1, a2)
        ring_hom_kernel_quotient_relation_respects_add(f)
        quotient_over_binary_op_projection_left(kernel_quotient_relation(f.hom), R.add, a1, a2, b)
    }
}

/// Addition in the kernel quotient respects related right representatives.
theorem ring_hom_kernel_quotient_add_projection_right[R: Ring, S: Ring](f: RingHom[R, S],
        a: R, b1: R, b2: R) {
    kernel_relation(f.hom, b1, b2) implies
    ring_hom_kernel_quotient_add(f,
        quotient_over_mk(kernel_quotient_relation(f.hom), a),
        quotient_over_mk(kernel_quotient_relation(f.hom), b1)
    ) = ring_hom_kernel_quotient_add(f,
        quotient_over_mk(kernel_quotient_relation(f.hom), a),
        quotient_over_mk(kernel_quotient_relation(f.hom), b2)
    )
} by {
    if kernel_relation(f.hom, b1, b2) {
        kernel_quotient_relation_rel(f.hom)
        kernel_quotient_relation(f.hom).rel(b1, b2)
        ring_hom_kernel_quotient_relation_respects_add(f)
        quotient_over_binary_op_projection_right(kernel_quotient_relation(f.hom), R.add, a, b1, b2)
    }
}

/// Zero is a left identity for addition on projected kernel quotient elements.
theorem ring_hom_kernel_quotient_add_zero_left_projection[R: Ring, S: Ring](f: RingHom[R, S], a: R) {
    ring_hom_kernel_quotient_add(f,
        ring_hom_kernel_quotient_zero(f),
        quotient_over_mk(kernel_quotient_relation(f.hom), a)
    ) = quotient_over_mk(kernel_quotient_relation(f.hom), a)
} by {
    ring_hom_kernel_quotient_relation_respects_add(f)
    R.0 + a = a
    R.add(R.0, a) = a
    quotient_over_binary_op_projection_identity_left(kernel_quotient_relation(f.hom), R.add, R.0, a)
}

/// Zero is a right identity for addition on projected kernel quotient elements.
theorem ring_hom_kernel_quotient_add_zero_right_projection[R: Ring, S: Ring](f: RingHom[R, S], a: R) {
    ring_hom_kernel_quotient_add(f,
        quotient_over_mk(kernel_quotient_relation(f.hom), a),
        ring_hom_kernel_quotient_zero(f)
    ) = quotient_over_mk(kernel_quotient_relation(f.hom), a)
} by {
    ring_hom_kernel_quotient_relation_respects_add(f)
    a + R.0 = a
    R.add(a, R.0) = a
    quotient_over_binary_op_projection_identity_right(kernel_quotient_relation(f.hom), R.add, R.0, a)
}

/// Multiplication on the kernel quotient by a ring homomorphism.
define ring_hom_kernel_quotient_mul[R: Ring, S: Ring](f: RingHom[R, S]) ->
    (QuotientOver[R], QuotientOver[R]) -> QuotientOver[R] {
    quotient_over_binary_op(R.mul)
}

/// The identity element on the kernel quotient by a ring homomorphism.
define ring_hom_kernel_quotient_one[R: Ring, S: Ring](f: RingHom[R, S]) -> QuotientOver[R] {
    quotient_over_const(kernel_quotient_relation(f.hom), R.1)
}

/// The identity element on the kernel quotient is the projection of the identity.
theorem ring_hom_kernel_quotient_one_projection[R: Ring, S: Ring](f: RingHom[R, S]) {
    ring_hom_kernel_quotient_one(f) = quotient_over_mk(kernel_quotient_relation(f.hom), R.1)
} by {
    quotient_over_const_projection(kernel_quotient_relation(f.hom), R.1)
}

/// Multiplication on the kernel quotient agrees with multiplication of representatives.
theorem ring_hom_kernel_quotient_mul_projection[R: Ring, S: Ring](f: RingHom[R, S], a: R, b: R) {
    ring_hom_kernel_quotient_mul(f,
        quotient_over_mk(kernel_quotient_relation(f.hom), a),
        quotient_over_mk(kernel_quotient_relation(f.hom), b)
    ) = quotient_over_mk(kernel_quotient_relation(f.hom), a * b)
} by {
    ring_hom_kernel_quotient_relation_respects_mul(f)
    quotient_over_binary_op_projection(kernel_quotient_relation(f.hom), R.mul, a, b)
    R.mul(a, b) = a * b
}

/// Multiplication in the kernel quotient respects related left representatives.
theorem ring_hom_kernel_quotient_mul_projection_left[R: Ring, S: Ring](f: RingHom[R, S],
        a1: R, a2: R, b: R) {
    kernel_relation(f.hom, a1, a2) implies
    ring_hom_kernel_quotient_mul(f,
        quotient_over_mk(kernel_quotient_relation(f.hom), a1),
        quotient_over_mk(kernel_quotient_relation(f.hom), b)
    ) = ring_hom_kernel_quotient_mul(f,
        quotient_over_mk(kernel_quotient_relation(f.hom), a2),
        quotient_over_mk(kernel_quotient_relation(f.hom), b)
    )
} by {
    if kernel_relation(f.hom, a1, a2) {
        kernel_quotient_relation_rel(f.hom)
        kernel_quotient_relation(f.hom).rel(a1, a2)
        ring_hom_kernel_quotient_relation_respects_mul(f)
        quotient_over_binary_op_projection_left(kernel_quotient_relation(f.hom), R.mul, a1, a2, b)
    }
}

/// Multiplication in the kernel quotient respects related right representatives.
theorem ring_hom_kernel_quotient_mul_projection_right[R: Ring, S: Ring](f: RingHom[R, S],
        a: R, b1: R, b2: R) {
    kernel_relation(f.hom, b1, b2) implies
    ring_hom_kernel_quotient_mul(f,
        quotient_over_mk(kernel_quotient_relation(f.hom), a),
        quotient_over_mk(kernel_quotient_relation(f.hom), b1)
    ) = ring_hom_kernel_quotient_mul(f,
        quotient_over_mk(kernel_quotient_relation(f.hom), a),
        quotient_over_mk(kernel_quotient_relation(f.hom), b2)
    )
} by {
    if kernel_relation(f.hom, b1, b2) {
        kernel_quotient_relation_rel(f.hom)
        kernel_quotient_relation(f.hom).rel(b1, b2)
        ring_hom_kernel_quotient_relation_respects_mul(f)
        quotient_over_binary_op_projection_right(kernel_quotient_relation(f.hom), R.mul, a, b1, b2)
    }
}

/// The identity is a left identity for multiplication on projected kernel quotient elements.
theorem ring_hom_kernel_quotient_mul_one_left_projection[R: Ring, S: Ring](f: RingHom[R, S], a: R) {
    ring_hom_kernel_quotient_mul(f,
        ring_hom_kernel_quotient_one(f),
        quotient_over_mk(kernel_quotient_relation(f.hom), a)
    ) = quotient_over_mk(kernel_quotient_relation(f.hom), a)
} by {
    ring_hom_kernel_quotient_relation_respects_mul(f)
    R.1 * a = a
    R.mul(R.1, a) = a
    quotient_over_binary_op_projection_identity_left(kernel_quotient_relation(f.hom), R.mul, R.1, a)
}

/// The identity is a right identity for multiplication on projected kernel quotient elements.
theorem ring_hom_kernel_quotient_mul_one_right_projection[R: Ring, S: Ring](f: RingHom[R, S], a: R) {
    ring_hom_kernel_quotient_mul(f,
        quotient_over_mk(kernel_quotient_relation(f.hom), a),
        ring_hom_kernel_quotient_one(f)
    ) = quotient_over_mk(kernel_quotient_relation(f.hom), a)
} by {
    ring_hom_kernel_quotient_relation_respects_mul(f)
    a * R.1 = a
    R.mul(a, R.1) = a
    quotient_over_binary_op_projection_identity_right(kernel_quotient_relation(f.hom), R.mul, R.1, a)
}

/// Powers on the kernel quotient by a ring homomorphism.
define ring_hom_kernel_quotient_pow[R: Ring, S: Ring](
    f: RingHom[R, S], q: QuotientOver[R], n: Nat
) -> QuotientOver[R] {
    match n {
        Nat.zero {
            ring_hom_kernel_quotient_one(f)
        }
        Nat.suc(k) {
            ring_hom_kernel_quotient_mul(f, q, ring_hom_kernel_quotient_pow(f, q, k))
        }
    }
}

/// The zeroth power on the ring-kernel quotient is the quotient identity.
theorem ring_hom_kernel_quotient_pow_zero_projection[R: Ring, S: Ring](
    f: RingHom[R, S], q: QuotientOver[R]
) {
    ring_hom_kernel_quotient_pow(f, q, Nat.0) = ring_hom_kernel_quotient_one(f)
} by {
}

/// The successor power on the ring-kernel quotient multiplies by the base.
theorem ring_hom_kernel_quotient_pow_suc_projection[R: Ring, S: Ring](
    f: RingHom[R, S], q: QuotientOver[R], n: Nat
) {
    ring_hom_kernel_quotient_pow(f, q, n.suc) =
        ring_hom_kernel_quotient_mul(f, q, ring_hom_kernel_quotient_pow(f, q, n))
}

/// The zeroth power on a projected ring representative is the projected identity.
theorem ring_hom_kernel_quotient_pow_mk_zero[R: Ring, S: Ring](f: RingHom[R, S], a: R) {
    ring_hom_kernel_quotient_pow(f, quotient_over_mk(kernel_quotient_relation(f.hom), a), Nat.0) =
        quotient_over_mk(kernel_quotient_relation(f.hom), a.pow(Nat.0))
} by {
    ring_hom_kernel_quotient_pow_zero_projection(f, quotient_over_mk(kernel_quotient_relation(f.hom), a))
    ring_hom_kernel_quotient_one_projection(f)
    a.pow(Nat.0) = R.1
}

/// Powers of projected ring representatives agree with projected powers.
theorem ring_hom_kernel_quotient_pow_mk[R: Ring, S: Ring](f: RingHom[R, S], a: R, n: Nat) {
    ring_hom_kernel_quotient_pow(f, quotient_over_mk(kernel_quotient_relation(f.hom), a), n) =
        quotient_over_mk(kernel_quotient_relation(f.hom), a.pow(n))
} by {
    define p(k: Nat) -> Bool {
        ring_hom_kernel_quotient_pow(f, quotient_over_mk(kernel_quotient_relation(f.hom), a), k) =
            quotient_over_mk(kernel_quotient_relation(f.hom), a.pow(k))
    }
    ring_hom_kernel_quotient_pow_mk_zero(f, a)
    p(Nat.0)
    forall(k: Nat) {
        if p(k) {
            ring_hom_kernel_quotient_pow_suc_projection(
                f, quotient_over_mk(kernel_quotient_relation(f.hom), a), k)
            ring_hom_kernel_quotient_pow(f,
                quotient_over_mk(kernel_quotient_relation(f.hom), a), k.suc) =
                ring_hom_kernel_quotient_mul(f,
                    quotient_over_mk(kernel_quotient_relation(f.hom), a),
                    ring_hom_kernel_quotient_pow(f,
                        quotient_over_mk(kernel_quotient_relation(f.hom), a), k))
            ring_hom_kernel_quotient_pow(f,
                quotient_over_mk(kernel_quotient_relation(f.hom), a), k) =
                quotient_over_mk(kernel_quotient_relation(f.hom), a.pow(k))
            ring_hom_kernel_quotient_mul(f,
                quotient_over_mk(kernel_quotient_relation(f.hom), a),
                ring_hom_kernel_quotient_pow(f,
                    quotient_over_mk(kernel_quotient_relation(f.hom), a), k)) =
                ring_hom_kernel_quotient_mul(f,
                    quotient_over_mk(kernel_quotient_relation(f.hom), a),
                    quotient_over_mk(kernel_quotient_relation(f.hom), a.pow(k)))
            ring_hom_kernel_quotient_mul_projection(f, a, a.pow(k))
            ring_hom_kernel_quotient_mul(f,
                quotient_over_mk(kernel_quotient_relation(f.hom), a),
                quotient_over_mk(kernel_quotient_relation(f.hom), a.pow(k))) =
                quotient_over_mk(kernel_quotient_relation(f.hom), a * a.pow(k))
            a.pow(k.suc) = a * a.pow(k)
            quotient_over_mk(kernel_quotient_relation(f.hom), a * a.pow(k)) =
                quotient_over_mk(kernel_quotient_relation(f.hom), a.pow(k.suc))
            p(k.suc)
        }
    }
    p(n)
}

/// Multiplying two projected ring-kernel powers adds their exponents.
theorem ring_hom_kernel_quotient_pow_add[R: Ring, S: Ring](
    f: RingHom[R, S], a: R, n: Nat, m: Nat
) {
    ring_hom_kernel_quotient_mul(f,
        ring_hom_kernel_quotient_pow(f, quotient_over_mk(kernel_quotient_relation(f.hom), a), n),
        ring_hom_kernel_quotient_pow(f, quotient_over_mk(kernel_quotient_relation(f.hom), a), m)) =
    ring_hom_kernel_quotient_pow(f, quotient_over_mk(kernel_quotient_relation(f.hom), a), n + m)
} by {
    ring_hom_kernel_quotient_pow_mk(f, a, n)
    ring_hom_kernel_quotient_pow_mk(f, a, m)
    ring_hom_kernel_quotient_mul(f,
        ring_hom_kernel_quotient_pow(f, quotient_over_mk(kernel_quotient_relation(f.hom), a), n),
        ring_hom_kernel_quotient_pow(f, quotient_over_mk(kernel_quotient_relation(f.hom), a), m)) =
    ring_hom_kernel_quotient_mul(f,
        quotient_over_mk(kernel_quotient_relation(f.hom), a.pow(n)),
        quotient_over_mk(kernel_quotient_relation(f.hom), a.pow(m)))
    ring_hom_kernel_quotient_mul_projection(f, a.pow(n), a.pow(m))
    ring_hom_kernel_quotient_mul(f,
        quotient_over_mk(kernel_quotient_relation(f.hom), a.pow(n)),
        quotient_over_mk(kernel_quotient_relation(f.hom), a.pow(m))) =
        quotient_over_mk(kernel_quotient_relation(f.hom), a.pow(n) * a.pow(m))
    ring_hom_kernel_quotient_pow_mk(f, a, n + m)
    ring_hom_kernel_quotient_pow(f, quotient_over_mk(kernel_quotient_relation(f.hom), a), n + m) =
        quotient_over_mk(kernel_quotient_relation(f.hom), a.pow(n + m))
    pow_add(a, n, m)
    a.pow(n) * a.pow(m) = a.pow(n + m)
    quotient_over_mk(kernel_quotient_relation(f.hom), a.pow(n) * a.pow(m)) =
        quotient_over_mk(kernel_quotient_relation(f.hom), a.pow(n + m))
}

/// Raising a projected ring-kernel power to a power multiplies the exponents.
theorem ring_hom_kernel_quotient_pow_pow[R: Ring, S: Ring](
    f: RingHom[R, S], a: R, n: Nat, m: Nat
) {
    ring_hom_kernel_quotient_pow(f,
        ring_hom_kernel_quotient_pow(f, quotient_over_mk(kernel_quotient_relation(f.hom), a), n),
        m) =
    ring_hom_kernel_quotient_pow(f, quotient_over_mk(kernel_quotient_relation(f.hom), a), n * m)
} by {
    ring_hom_kernel_quotient_pow_mk(f, a, n)
    ring_hom_kernel_quotient_pow(f, quotient_over_mk(kernel_quotient_relation(f.hom), a), n) =
        quotient_over_mk(kernel_quotient_relation(f.hom), a.pow(n))
    ring_hom_kernel_quotient_pow(f,
        ring_hom_kernel_quotient_pow(f, quotient_over_mk(kernel_quotient_relation(f.hom), a), n),
        m) =
    ring_hom_kernel_quotient_pow(f,
        quotient_over_mk(kernel_quotient_relation(f.hom), a.pow(n)),
        m)
    ring_hom_kernel_quotient_pow_mk(f, a.pow(n), m)
    ring_hom_kernel_quotient_pow(f,
        quotient_over_mk(kernel_quotient_relation(f.hom), a.pow(n)),
        m) =
        quotient_over_mk(kernel_quotient_relation(f.hom), a.pow(n).pow(m))
    ring_hom_kernel_quotient_pow_mk(f, a, n * m)
    ring_hom_kernel_quotient_pow(f, quotient_over_mk(kernel_quotient_relation(f.hom), a), n * m) =
        quotient_over_mk(kernel_quotient_relation(f.hom), a.pow(n * m))
    pow_pow(a, n, m)
    a.pow(n).pow(m) = a.pow(n * m)
    quotient_over_mk(kernel_quotient_relation(f.hom), a.pow(n).pow(m)) =
        quotient_over_mk(kernel_quotient_relation(f.hom), a.pow(n * m))
}

/// Quotient one raised to a natural power is quotient one in a ring-kernel quotient.
theorem ring_hom_kernel_quotient_one_pow[R: Ring, S: Ring](f: RingHom[R, S], n: Nat) {
    ring_hom_kernel_quotient_pow(f, ring_hom_kernel_quotient_one(f), n) =
        ring_hom_kernel_quotient_one(f)
} by {
    ring_hom_kernel_quotient_one_projection(f)
    ring_hom_kernel_quotient_pow(f, ring_hom_kernel_quotient_one(f), n) =
    ring_hom_kernel_quotient_pow(f, quotient_over_mk(kernel_quotient_relation(f.hom), R.1), n)
    ring_hom_kernel_quotient_pow_mk(f, R.1, n)
    ring_hom_kernel_quotient_pow(f, quotient_over_mk(kernel_quotient_relation(f.hom), R.1), n) =
        quotient_over_mk(kernel_quotient_relation(f.hom), R.1.pow(n))
    one_pow[R](n)
    R.1.pow(n) = R.1
    quotient_over_mk(kernel_quotient_relation(f.hom), R.1.pow(n)) =
        quotient_over_mk(kernel_quotient_relation(f.hom), R.1)
}

/// Negation on the kernel quotient by a ring homomorphism.
define ring_hom_kernel_quotient_neg[R: Ring, S: Ring](f: RingHom[R, S]) ->
    QuotientOver[R] -> QuotientOver[R] {
    quotient_over_unary_op(kernel_quotient_relation(f.hom), R.neg)
}

/// Negation on the kernel quotient agrees with negation of representatives.
theorem ring_hom_kernel_quotient_neg_projection[R: Ring, S: Ring](f: RingHom[R, S], a: R) {
    ring_hom_kernel_quotient_neg(f, quotient_over_mk(kernel_quotient_relation(f.hom), a)) =
        quotient_over_mk(kernel_quotient_relation(f.hom), -a)
} by {
    ring_hom_kernel_quotient_relation_respects_neg(f)
    quotient_over_unary_op_projection(kernel_quotient_relation(f.hom), R.neg, a)
    R.neg(a) = -a
}

/// Negation on the kernel quotient respects related representatives.
theorem ring_hom_kernel_quotient_neg_projection_compatible[R: Ring, S: Ring](f: RingHom[R, S],
        a: R, b: R) {
    kernel_relation(f.hom, a, b) implies
    ring_hom_kernel_quotient_neg(f, quotient_over_mk(kernel_quotient_relation(f.hom), a)) =
        ring_hom_kernel_quotient_neg(f, quotient_over_mk(kernel_quotient_relation(f.hom), b))
} by {
    if kernel_relation(f.hom, a, b) {
        kernel_quotient_relation_rel(f.hom)
        kernel_quotient_relation(f.hom).rel(a, b)
        ring_hom_kernel_quotient_relation_respects_neg(f)
        quotient_over_unary_op_projection_compatible(kernel_quotient_relation(f.hom), R.neg, a, b)
    }
}

/// Addition on the ring kernel quotient is associative.
theorem ring_hom_kernel_quotient_add_assoc[R: Ring, S: Ring](f: RingHom[R, S], a: R, b: R, c: R) {
    ring_hom_kernel_quotient_add(f,
        ring_hom_kernel_quotient_add(f,
            quotient_over_mk(kernel_quotient_relation(f.hom), a),
            quotient_over_mk(kernel_quotient_relation(f.hom), b)),
        quotient_over_mk(kernel_quotient_relation(f.hom), c)) =
    ring_hom_kernel_quotient_add(f,
        quotient_over_mk(kernel_quotient_relation(f.hom), a),
        ring_hom_kernel_quotient_add(f,
            quotient_over_mk(kernel_quotient_relation(f.hom), b),
            quotient_over_mk(kernel_quotient_relation(f.hom), c)))
} by {
    ring_hom_kernel_quotient_add_projection(f, a, b)
    ring_hom_kernel_quotient_add(f,
        quotient_over_mk(kernel_quotient_relation(f.hom), a),
        quotient_over_mk(kernel_quotient_relation(f.hom), b)) =
        quotient_over_mk(kernel_quotient_relation(f.hom), a + b)
    ring_hom_kernel_quotient_add(f,
        ring_hom_kernel_quotient_add(f,
            quotient_over_mk(kernel_quotient_relation(f.hom), a),
            quotient_over_mk(kernel_quotient_relation(f.hom), b)),
        quotient_over_mk(kernel_quotient_relation(f.hom), c)) =
    ring_hom_kernel_quotient_add(f,
        quotient_over_mk(kernel_quotient_relation(f.hom), a + b),
        quotient_over_mk(kernel_quotient_relation(f.hom), c))
    ring_hom_kernel_quotient_add_projection(f, a + b, c)
    ring_hom_kernel_quotient_add(f,
        quotient_over_mk(kernel_quotient_relation(f.hom), a + b),
        quotient_over_mk(kernel_quotient_relation(f.hom), c)) =
        quotient_over_mk(kernel_quotient_relation(f.hom), (a + b) + c)
    ring_hom_kernel_quotient_add_projection(f, b, c)
    ring_hom_kernel_quotient_add(f,
        quotient_over_mk(kernel_quotient_relation(f.hom), b),
        quotient_over_mk(kernel_quotient_relation(f.hom), c)) =
        quotient_over_mk(kernel_quotient_relation(f.hom), b + c)
    ring_hom_kernel_quotient_add(f,
        quotient_over_mk(kernel_quotient_relation(f.hom), a),
        ring_hom_kernel_quotient_add(f,
            quotient_over_mk(kernel_quotient_relation(f.hom), b),
            quotient_over_mk(kernel_quotient_relation(f.hom), c))) =
    ring_hom_kernel_quotient_add(f,
        quotient_over_mk(kernel_quotient_relation(f.hom), a),
        quotient_over_mk(kernel_quotient_relation(f.hom), b + c))
    ring_hom_kernel_quotient_add_projection(f, a, b + c)
    ring_hom_kernel_quotient_add(f,
        quotient_over_mk(kernel_quotient_relation(f.hom), a),
        quotient_over_mk(kernel_quotient_relation(f.hom), b + c)) =
        quotient_over_mk(kernel_quotient_relation(f.hom), a + (b + c))
    (a + b) + c = a + (b + c)
    quotient_over_mk(kernel_quotient_relation(f.hom), (a + b) + c) =
        quotient_over_mk(kernel_quotient_relation(f.hom), a + (b + c))
    ring_hom_kernel_quotient_add(f,
        ring_hom_kernel_quotient_add(f,
            quotient_over_mk(kernel_quotient_relation(f.hom), a),
            quotient_over_mk(kernel_quotient_relation(f.hom), b)),
        quotient_over_mk(kernel_quotient_relation(f.hom), c)) =
        quotient_over_mk(kernel_quotient_relation(f.hom), (a + b) + c)
    ring_hom_kernel_quotient_add(f,
        quotient_over_mk(kernel_quotient_relation(f.hom), a),
        ring_hom_kernel_quotient_add(f,
            quotient_over_mk(kernel_quotient_relation(f.hom), b),
            quotient_over_mk(kernel_quotient_relation(f.hom), c))) =
        quotient_over_mk(kernel_quotient_relation(f.hom), a + (b + c))
}

/// The quotient zero is a left identity for ring quotient addition.
theorem ring_hom_kernel_quotient_zero_add[R: Ring, S: Ring](f: RingHom[R, S], a: R) {
    ring_hom_kernel_quotient_add(f,
        quotient_over_mk(kernel_quotient_relation(f.hom), R.0),
        quotient_over_mk(kernel_quotient_relation(f.hom), a)) =
        quotient_over_mk(kernel_quotient_relation(f.hom), a)
} by {
    ring_hom_kernel_quotient_add_projection(f, R.0, a)
    R.0 + a = a
}

/// The quotient zero is a right identity for ring quotient addition.
theorem ring_hom_kernel_quotient_add_zero[R: Ring, S: Ring](f: RingHom[R, S], a: R) {
    ring_hom_kernel_quotient_add(f,
        quotient_over_mk(kernel_quotient_relation(f.hom), a),
        quotient_over_mk(kernel_quotient_relation(f.hom), R.0)) =
        quotient_over_mk(kernel_quotient_relation(f.hom), a)
} by {
    ring_hom_kernel_quotient_add_projection(f, a, R.0)
    a + R.0 = a
}

/// The quotient negation is a left inverse for ring quotient addition.
theorem ring_hom_kernel_quotient_neg_add[R: Ring, S: Ring](f: RingHom[R, S], a: R) {
    ring_hom_kernel_quotient_add(f,
        ring_hom_kernel_quotient_neg(f, quotient_over_mk(kernel_quotient_relation(f.hom), a)),
        quotient_over_mk(kernel_quotient_relation(f.hom), a)) =
        quotient_over_mk(kernel_quotient_relation(f.hom), R.0)
} by {
    ring_hom_kernel_quotient_neg_projection(f, a)
    ring_hom_kernel_quotient_neg(f, quotient_over_mk(kernel_quotient_relation(f.hom), a)) =
        quotient_over_mk(kernel_quotient_relation(f.hom), -a)
    ring_hom_kernel_quotient_add(f,
        ring_hom_kernel_quotient_neg(f, quotient_over_mk(kernel_quotient_relation(f.hom), a)),
        quotient_over_mk(kernel_quotient_relation(f.hom), a)) =
    ring_hom_kernel_quotient_add(f,
        quotient_over_mk(kernel_quotient_relation(f.hom), -a),
        quotient_over_mk(kernel_quotient_relation(f.hom), a))
    ring_hom_kernel_quotient_add_projection(f, -a, a)
    ring_hom_kernel_quotient_add(f,
        quotient_over_mk(kernel_quotient_relation(f.hom), -a),
        quotient_over_mk(kernel_quotient_relation(f.hom), a)) =
        quotient_over_mk(kernel_quotient_relation(f.hom), -a + a)
    -a + a = R.0
    quotient_over_mk(kernel_quotient_relation(f.hom), -a + a) =
        quotient_over_mk(kernel_quotient_relation(f.hom), R.0)
    ring_hom_kernel_quotient_add(f,
        ring_hom_kernel_quotient_neg(f, quotient_over_mk(kernel_quotient_relation(f.hom), a)),
        quotient_over_mk(kernel_quotient_relation(f.hom), a)) =
        quotient_over_mk(kernel_quotient_relation(f.hom), R.0)
}

/// The quotient negation is a right inverse for ring quotient addition.
theorem ring_hom_kernel_quotient_add_neg[R: Ring, S: Ring](f: RingHom[R, S], a: R) {
    ring_hom_kernel_quotient_add(f,
        quotient_over_mk(kernel_quotient_relation(f.hom), a),
        ring_hom_kernel_quotient_neg(f, quotient_over_mk(kernel_quotient_relation(f.hom), a))) =
        quotient_over_mk(kernel_quotient_relation(f.hom), R.0)
} by {
    ring_hom_kernel_quotient_neg_projection(f, a)
    ring_hom_kernel_quotient_neg(f, quotient_over_mk(kernel_quotient_relation(f.hom), a)) =
        quotient_over_mk(kernel_quotient_relation(f.hom), -a)
    ring_hom_kernel_quotient_add(f,
        quotient_over_mk(kernel_quotient_relation(f.hom), a),
        ring_hom_kernel_quotient_neg(f, quotient_over_mk(kernel_quotient_relation(f.hom), a))) =
    ring_hom_kernel_quotient_add(f,
        quotient_over_mk(kernel_quotient_relation(f.hom), a),
        quotient_over_mk(kernel_quotient_relation(f.hom), -a))
    ring_hom_kernel_quotient_add_projection(f, a, -a)
    ring_hom_kernel_quotient_add(f,
        quotient_over_mk(kernel_quotient_relation(f.hom), a),
        quotient_over_mk(kernel_quotient_relation(f.hom), -a)) =
        quotient_over_mk(kernel_quotient_relation(f.hom), a + -a)
    a + -a = R.0
    quotient_over_mk(kernel_quotient_relation(f.hom), a + -a) =
        quotient_over_mk(kernel_quotient_relation(f.hom), R.0)
    ring_hom_kernel_quotient_add(f,
        quotient_over_mk(kernel_quotient_relation(f.hom), a),
        ring_hom_kernel_quotient_neg(f, quotient_over_mk(kernel_quotient_relation(f.hom), a))) =
        quotient_over_mk(kernel_quotient_relation(f.hom), R.0)
}

/// Subtracting quotient zero fixes a ring-kernel quotient representative.
theorem ring_hom_kernel_quotient_sub_zero[R: Ring, S: Ring](f: RingHom[R, S], a: R) {
    ring_hom_kernel_quotient_sub(f,
        quotient_over_mk(kernel_quotient_relation(f.hom), a),
        quotient_over_mk(kernel_quotient_relation(f.hom), R.0)) =
        quotient_over_mk(kernel_quotient_relation(f.hom), a)
} by {
    ring_hom_kernel_quotient_sub_projection(f, a, R.0)
    a - R.0 = a + -R.0
    -R.0 = R.0
    a + R.0 = a
}

/// Subtracting a ring-kernel quotient representative from zero gives quotient negation.
theorem ring_hom_kernel_quotient_zero_sub[R: Ring, S: Ring](f: RingHom[R, S], a: R) {
    ring_hom_kernel_quotient_sub(f,
        quotient_over_mk(kernel_quotient_relation(f.hom), R.0),
        quotient_over_mk(kernel_quotient_relation(f.hom), a)) =
        ring_hom_kernel_quotient_neg(f,
            quotient_over_mk(kernel_quotient_relation(f.hom), a))
} by {
    ring_hom_kernel_quotient_sub_projection(f, R.0, a)
    ring_hom_kernel_quotient_neg_projection(f, a)
    R.0 - a = R.0 + -a
    R.0 + -a = -a
}

/// Subtracting a ring-kernel quotient representative from itself gives quotient zero.
theorem ring_hom_kernel_quotient_sub_self[R: Ring, S: Ring](f: RingHom[R, S], a: R) {
    ring_hom_kernel_quotient_sub(f,
        quotient_over_mk(kernel_quotient_relation(f.hom), a),
        quotient_over_mk(kernel_quotient_relation(f.hom), a)) =
        quotient_over_mk(kernel_quotient_relation(f.hom), R.0)
} by {
    ring_hom_kernel_quotient_sub_projection(f, a, a)
    a - a = a + -a
    a + -a = R.0
}

/// Ring-kernel quotient subtraction is addition of quotient negation.
theorem ring_hom_kernel_quotient_sub_eq_add_neg[R: Ring, S: Ring](f: RingHom[R, S], a: R, b: R) {
    ring_hom_kernel_quotient_sub(f,
        quotient_over_mk(kernel_quotient_relation(f.hom), a),
        quotient_over_mk(kernel_quotient_relation(f.hom), b)) =
    ring_hom_kernel_quotient_add(f,
        quotient_over_mk(kernel_quotient_relation(f.hom), a),
        ring_hom_kernel_quotient_neg(f,
            quotient_over_mk(kernel_quotient_relation(f.hom), b)))
} by {
    ring_hom_kernel_quotient_sub_projection(f, a, b)
    ring_hom_kernel_quotient_neg_projection(f, b)
    ring_hom_kernel_quotient_add(f,
        quotient_over_mk(kernel_quotient_relation(f.hom), a),
        ring_hom_kernel_quotient_neg(f,
            quotient_over_mk(kernel_quotient_relation(f.hom), b))) =
    ring_hom_kernel_quotient_add(f,
        quotient_over_mk(kernel_quotient_relation(f.hom), a),
        quotient_over_mk(kernel_quotient_relation(f.hom), -b))
    ring_hom_kernel_quotient_add_projection(f, a, -b)
    a - b = a + -b
}

/// A ring-kernel quotient negation cancels a matching left summand.
theorem ring_hom_kernel_quotient_neg_add_cancel_left[R: Ring, S: Ring](
    f: RingHom[R, S], a: R, b: R
) {
    ring_hom_kernel_quotient_add(f,
        ring_hom_kernel_quotient_neg(f,
            quotient_over_mk(kernel_quotient_relation(f.hom), a)),
        ring_hom_kernel_quotient_add(f,
            quotient_over_mk(kernel_quotient_relation(f.hom), a),
            quotient_over_mk(kernel_quotient_relation(f.hom), b))) =
        quotient_over_mk(kernel_quotient_relation(f.hom), b)
} by {
    ring_hom_kernel_quotient_add_projection(f, a, b)
    ring_hom_kernel_quotient_neg_projection(f, a)
    ring_hom_kernel_quotient_add(f,
        ring_hom_kernel_quotient_neg(f,
            quotient_over_mk(kernel_quotient_relation(f.hom), a)),
        ring_hom_kernel_quotient_add(f,
            quotient_over_mk(kernel_quotient_relation(f.hom), a),
            quotient_over_mk(kernel_quotient_relation(f.hom), b))) =
    ring_hom_kernel_quotient_add(f,
        quotient_over_mk(kernel_quotient_relation(f.hom), -a),
        quotient_over_mk(kernel_quotient_relation(f.hom), a + b))
    ring_hom_kernel_quotient_add_projection(f, -a, a + b)
    -a + (a + b) = (-a + a) + b
    -a + a = R.0
    (-a + a) + b = R.0 + b
    R.0 + b = b
    quotient_over_mk(kernel_quotient_relation(f.hom), -a + (a + b)) =
        quotient_over_mk(kernel_quotient_relation(f.hom), b)
}

/// A ring-kernel quotient negation cancels a matching right summand.
theorem ring_hom_kernel_quotient_add_neg_cancel_right[R: Ring, S: Ring](
    f: RingHom[R, S], a: R, b: R
) {
    ring_hom_kernel_quotient_add(f,
        ring_hom_kernel_quotient_add(f,
            quotient_over_mk(kernel_quotient_relation(f.hom), b),
            quotient_over_mk(kernel_quotient_relation(f.hom), a)),
        ring_hom_kernel_quotient_neg(f,
            quotient_over_mk(kernel_quotient_relation(f.hom), a))) =
        quotient_over_mk(kernel_quotient_relation(f.hom), b)
} by {
    ring_hom_kernel_quotient_add_projection(f, b, a)
    ring_hom_kernel_quotient_neg_projection(f, a)
    ring_hom_kernel_quotient_add(f,
        ring_hom_kernel_quotient_add(f,
            quotient_over_mk(kernel_quotient_relation(f.hom), b),
            quotient_over_mk(kernel_quotient_relation(f.hom), a)),
        ring_hom_kernel_quotient_neg(f,
            quotient_over_mk(kernel_quotient_relation(f.hom), a))) =
    ring_hom_kernel_quotient_add(f,
        quotient_over_mk(kernel_quotient_relation(f.hom), b + a),
        quotient_over_mk(kernel_quotient_relation(f.hom), -a))
    ring_hom_kernel_quotient_add_projection(f, b + a, -a)
    (b + a) + -a = b + (a + -a)
    a + -a = R.0
    b + (a + -a) = b + R.0
    b + R.0 = b
    quotient_over_mk(kernel_quotient_relation(f.hom), (b + a) + -a) =
        quotient_over_mk(kernel_quotient_relation(f.hom), b)
}

/// Subtracting the right summand of a ring-kernel quotient sum cancels it.
theorem ring_hom_kernel_quotient_add_sub_cancel_right[R: Ring, S: Ring](
    f: RingHom[R, S], a: R, b: R
) {
    ring_hom_kernel_quotient_sub(f,
        ring_hom_kernel_quotient_add(f,
            quotient_over_mk(kernel_quotient_relation(f.hom), a),
            quotient_over_mk(kernel_quotient_relation(f.hom), b)),
        quotient_over_mk(kernel_quotient_relation(f.hom), b)) =
        quotient_over_mk(kernel_quotient_relation(f.hom), a)
} by {
    ring_hom_kernel_quotient_sub_eq_add_neg(f, a + b, b)
    ring_hom_kernel_quotient_add_projection(f, a, b)
    ring_hom_kernel_quotient_add_neg_cancel_right(f, b, a)
}

/// Subtracting the left summand of a ring-kernel quotient sum cancels it.
theorem ring_hom_kernel_quotient_add_sub_cancel_left[R: Ring, S: Ring](
    f: RingHom[R, S], a: R, b: R
) {
    ring_hom_kernel_quotient_sub(f,
        ring_hom_kernel_quotient_add(f,
            quotient_over_mk(kernel_quotient_relation(f.hom), a),
            quotient_over_mk(kernel_quotient_relation(f.hom), b)),
        quotient_over_mk(kernel_quotient_relation(f.hom), a)) =
        quotient_over_mk(kernel_quotient_relation(f.hom), b)
} by {
    ring_hom_kernel_quotient_add_projection(f, a, b)
    ring_hom_kernel_quotient_sub_projection(f, a + b, a)
    a + b - a = a + b + -a
    a + b = b + a
    a + b + -a = (b + a) + -a
    (b + a) + -a = b + (a + -a)
    a + -a = R.0
    b + R.0 = b
    quotient_over_mk(kernel_quotient_relation(f.hom), a + b - a) =
        quotient_over_mk(kernel_quotient_relation(f.hom), b)
}

/// The negation of ring-kernel quotient zero is quotient zero.
theorem ring_hom_kernel_quotient_neg_zero[R: Ring, S: Ring](f: RingHom[R, S]) {
    ring_hom_kernel_quotient_neg(f, ring_hom_kernel_quotient_zero(f)) =
        ring_hom_kernel_quotient_zero(f)
} by {
    ring_hom_kernel_quotient_zero_projection(f)
    ring_hom_kernel_quotient_neg(f, ring_hom_kernel_quotient_zero(f)) =
        ring_hom_kernel_quotient_neg(f, quotient_over_mk(kernel_quotient_relation(f.hom), R.0))
    ring_hom_kernel_quotient_neg_projection(f, R.0)
    ring_hom_kernel_quotient_neg(f, quotient_over_mk(kernel_quotient_relation(f.hom), R.0)) =
        quotient_over_mk(kernel_quotient_relation(f.hom), -R.0)
    -R.0 = R.0
    quotient_over_mk(kernel_quotient_relation(f.hom), -R.0) =
        quotient_over_mk(kernel_quotient_relation(f.hom), R.0)
}

/// Negating twice fixes a ring-kernel quotient representative.
theorem ring_hom_kernel_quotient_neg_neg[R: Ring, S: Ring](f: RingHom[R, S], a: R) {
    ring_hom_kernel_quotient_neg(f,
        ring_hom_kernel_quotient_neg(f, quotient_over_mk(kernel_quotient_relation(f.hom), a))) =
        quotient_over_mk(kernel_quotient_relation(f.hom), a)
} by {
    ring_hom_kernel_quotient_neg_projection(f, a)
    ring_hom_kernel_quotient_neg(f, quotient_over_mk(kernel_quotient_relation(f.hom), a)) =
        quotient_over_mk(kernel_quotient_relation(f.hom), -a)
    ring_hom_kernel_quotient_neg(f,
        ring_hom_kernel_quotient_neg(f, quotient_over_mk(kernel_quotient_relation(f.hom), a))) =
        ring_hom_kernel_quotient_neg(f, quotient_over_mk(kernel_quotient_relation(f.hom), -a))
    ring_hom_kernel_quotient_neg_projection(f, -a)
    ring_hom_kernel_quotient_neg(f, quotient_over_mk(kernel_quotient_relation(f.hom), -a)) =
        quotient_over_mk(kernel_quotient_relation(f.hom), -(-a))
    -(-a) = a
    quotient_over_mk(kernel_quotient_relation(f.hom), -(-a)) =
        quotient_over_mk(kernel_quotient_relation(f.hom), a)
}

/// Negation distributes over ring-kernel quotient addition.
theorem ring_hom_kernel_quotient_neg_add_projection[R: Ring, S: Ring](f: RingHom[R, S], a: R, b: R) {
    ring_hom_kernel_quotient_neg(f,
        ring_hom_kernel_quotient_add(f,
            quotient_over_mk(kernel_quotient_relation(f.hom), a),
            quotient_over_mk(kernel_quotient_relation(f.hom), b))) =
    ring_hom_kernel_quotient_add(f,
        ring_hom_kernel_quotient_neg(f, quotient_over_mk(kernel_quotient_relation(f.hom), a)),
        ring_hom_kernel_quotient_neg(f, quotient_over_mk(kernel_quotient_relation(f.hom), b)))
} by {
    ring_hom_kernel_quotient_add_projection(f, a, b)
    ring_hom_kernel_quotient_neg_projection(f, a + b)
    ring_hom_kernel_quotient_neg_projection(f, a)
    ring_hom_kernel_quotient_neg_projection(f, b)
    ring_hom_kernel_quotient_add_projection(f, -a, -b)
    -(a + b) = -a + -b
    quotient_over_mk(kernel_quotient_relation(f.hom), -(a + b)) =
        quotient_over_mk(kernel_quotient_relation(f.hom), -a + -b)
}

/// Negation sends ring-kernel quotient subtraction to the opposite subtraction.
theorem ring_hom_kernel_quotient_neg_sub_projection[R: Ring, S: Ring](f: RingHom[R, S], a: R, b: R) {
    ring_hom_kernel_quotient_neg(f,
        ring_hom_kernel_quotient_sub(f,
            quotient_over_mk(kernel_quotient_relation(f.hom), a),
            quotient_over_mk(kernel_quotient_relation(f.hom), b))) =
    ring_hom_kernel_quotient_sub(f,
        quotient_over_mk(kernel_quotient_relation(f.hom), b),
        quotient_over_mk(kernel_quotient_relation(f.hom), a))
} by {
    ring_hom_kernel_quotient_sub_projection(f, a, b)
    ring_hom_kernel_quotient_neg_projection(f, a - b)
    ring_hom_kernel_quotient_sub_projection(f, b, a)
    a - b = a + -b
    -(a - b) = -(a + -b)
    -(a + -b) = -a + --b
    --b = b
    -a + b = b + -a
    b + -a = b - a
    -(a - b) = b - a
    quotient_over_mk(kernel_quotient_relation(f.hom), -(a - b)) =
        quotient_over_mk(kernel_quotient_relation(f.hom), b - a)
}

/// Subtracting a negated ring-kernel quotient representative is quotient addition.
theorem ring_hom_kernel_quotient_sub_neg_projection[R: Ring, S: Ring](f: RingHom[R, S], a: R, b: R) {
    ring_hom_kernel_quotient_sub(f,
        quotient_over_mk(kernel_quotient_relation(f.hom), a),
        ring_hom_kernel_quotient_neg(f, quotient_over_mk(kernel_quotient_relation(f.hom), b))) =
    ring_hom_kernel_quotient_add(f,
        quotient_over_mk(kernel_quotient_relation(f.hom), a),
        quotient_over_mk(kernel_quotient_relation(f.hom), b))
} by {
    ring_hom_kernel_quotient_neg_projection(f, b)
    ring_hom_kernel_quotient_sub_projection(f, a, -b)
    ring_hom_kernel_quotient_add_projection(f, a, b)
    a - -b = a + b
    quotient_over_mk(kernel_quotient_relation(f.hom), a - -b) =
        quotient_over_mk(kernel_quotient_relation(f.hom), a + b)
}

/// A ring-kernel quotient product with a negated left representative is the negation of the product.
theorem ring_hom_kernel_quotient_neg_mul[R: Ring, S: Ring](f: RingHom[R, S], a: R, b: R) {
    ring_hom_kernel_quotient_mul(f,
        ring_hom_kernel_quotient_neg(f, quotient_over_mk(kernel_quotient_relation(f.hom), a)),
        quotient_over_mk(kernel_quotient_relation(f.hom), b)) =
    ring_hom_kernel_quotient_neg(f,
        ring_hom_kernel_quotient_mul(f,
            quotient_over_mk(kernel_quotient_relation(f.hom), a),
            quotient_over_mk(kernel_quotient_relation(f.hom), b)))
} by {
    ring_hom_kernel_quotient_neg_projection(f, a)
    ring_hom_kernel_quotient_mul_projection(f, -a, b)
    ring_hom_kernel_quotient_mul_projection(f, a, b)
    ring_hom_kernel_quotient_neg_projection(f, a * b)
    -a * b = -(a * b)
    quotient_over_mk(kernel_quotient_relation(f.hom), -a * b) =
        quotient_over_mk(kernel_quotient_relation(f.hom), -(a * b))
}

/// A ring-kernel quotient product with a negated right representative is the negation of the product.
theorem ring_hom_kernel_quotient_mul_neg[R: Ring, S: Ring](f: RingHom[R, S], a: R, b: R) {
    ring_hom_kernel_quotient_mul(f,
        quotient_over_mk(kernel_quotient_relation(f.hom), a),
        ring_hom_kernel_quotient_neg(f, quotient_over_mk(kernel_quotient_relation(f.hom), b))) =
    ring_hom_kernel_quotient_neg(f,
        ring_hom_kernel_quotient_mul(f,
            quotient_over_mk(kernel_quotient_relation(f.hom), a),
            quotient_over_mk(kernel_quotient_relation(f.hom), b)))
} by {
    ring_hom_kernel_quotient_neg_projection(f, b)
    ring_hom_kernel_quotient_mul_projection(f, a, -b)
    ring_hom_kernel_quotient_mul_projection(f, a, b)
    ring_hom_kernel_quotient_neg_projection(f, a * b)
    a * -b = -(a * b)
    quotient_over_mk(kernel_quotient_relation(f.hom), a * -b) =
        quotient_over_mk(kernel_quotient_relation(f.hom), -(a * b))
}

/// The product of two negated ring-kernel quotient representatives is the product.
theorem ring_hom_kernel_quotient_neg_mul_neg[R: Ring, S: Ring](f: RingHom[R, S], a: R, b: R) {
    ring_hom_kernel_quotient_mul(f,
        ring_hom_kernel_quotient_neg(f, quotient_over_mk(kernel_quotient_relation(f.hom), a)),
        ring_hom_kernel_quotient_neg(f, quotient_over_mk(kernel_quotient_relation(f.hom), b))) =
    ring_hom_kernel_quotient_mul(f,
        quotient_over_mk(kernel_quotient_relation(f.hom), a),
        quotient_over_mk(kernel_quotient_relation(f.hom), b))
} by {
    ring_hom_kernel_quotient_neg_projection(f, a)
    ring_hom_kernel_quotient_neg_projection(f, b)
    ring_hom_kernel_quotient_mul_projection(f, -a, -b)
    ring_hom_kernel_quotient_mul_projection(f, a, b)
    -a * -b = a * b
    quotient_over_mk(kernel_quotient_relation(f.hom), -a * -b) =
        quotient_over_mk(kernel_quotient_relation(f.hom), a * b)
}

/// Addition on the ring kernel quotient is commutative.
theorem ring_hom_kernel_quotient_add_comm[R: Ring, S: Ring](f: RingHom[R, S], a: R, b: R) {
    ring_hom_kernel_quotient_add(f,
        quotient_over_mk(kernel_quotient_relation(f.hom), a),
        quotient_over_mk(kernel_quotient_relation(f.hom), b)) =
    ring_hom_kernel_quotient_add(f,
        quotient_over_mk(kernel_quotient_relation(f.hom), b),
        quotient_over_mk(kernel_quotient_relation(f.hom), a))
} by {
    ring_hom_kernel_quotient_add_projection(f, a, b)
    ring_hom_kernel_quotient_add_projection(f, b, a)
    a + b = b + a
    quotient_over_mk(kernel_quotient_relation(f.hom), a + b) =
        quotient_over_mk(kernel_quotient_relation(f.hom), b + a)
}

/// Multiplication on the ring kernel quotient is associative.
theorem ring_hom_kernel_quotient_mul_assoc[R: Ring, S: Ring](f: RingHom[R, S], a: R, b: R, c: R) {
    ring_hom_kernel_quotient_mul(f,
        ring_hom_kernel_quotient_mul(f,
            quotient_over_mk(kernel_quotient_relation(f.hom), a),
            quotient_over_mk(kernel_quotient_relation(f.hom), b)),
        quotient_over_mk(kernel_quotient_relation(f.hom), c)) =
    ring_hom_kernel_quotient_mul(f,
        quotient_over_mk(kernel_quotient_relation(f.hom), a),
        ring_hom_kernel_quotient_mul(f,
            quotient_over_mk(kernel_quotient_relation(f.hom), b),
            quotient_over_mk(kernel_quotient_relation(f.hom), c)))
} by {
    ring_hom_kernel_quotient_mul_projection(f, a, b)
    ring_hom_kernel_quotient_mul(f,
        quotient_over_mk(kernel_quotient_relation(f.hom), a),
        quotient_over_mk(kernel_quotient_relation(f.hom), b)) =
        quotient_over_mk(kernel_quotient_relation(f.hom), a * b)
    ring_hom_kernel_quotient_mul(f,
        ring_hom_kernel_quotient_mul(f,
            quotient_over_mk(kernel_quotient_relation(f.hom), a),
            quotient_over_mk(kernel_quotient_relation(f.hom), b)),
        quotient_over_mk(kernel_quotient_relation(f.hom), c)) =
    ring_hom_kernel_quotient_mul(f,
        quotient_over_mk(kernel_quotient_relation(f.hom), a * b),
        quotient_over_mk(kernel_quotient_relation(f.hom), c))
    ring_hom_kernel_quotient_mul_projection(f, a * b, c)
    ring_hom_kernel_quotient_mul(f,
        quotient_over_mk(kernel_quotient_relation(f.hom), a * b),
        quotient_over_mk(kernel_quotient_relation(f.hom), c)) =
        quotient_over_mk(kernel_quotient_relation(f.hom), (a * b) * c)
    ring_hom_kernel_quotient_mul_projection(f, b, c)
    ring_hom_kernel_quotient_mul(f,
        quotient_over_mk(kernel_quotient_relation(f.hom), b),
        quotient_over_mk(kernel_quotient_relation(f.hom), c)) =
        quotient_over_mk(kernel_quotient_relation(f.hom), b * c)
    ring_hom_kernel_quotient_mul(f,
        quotient_over_mk(kernel_quotient_relation(f.hom), a),
        ring_hom_kernel_quotient_mul(f,
            quotient_over_mk(kernel_quotient_relation(f.hom), b),
            quotient_over_mk(kernel_quotient_relation(f.hom), c))) =
    ring_hom_kernel_quotient_mul(f,
        quotient_over_mk(kernel_quotient_relation(f.hom), a),
        quotient_over_mk(kernel_quotient_relation(f.hom), b * c))
    ring_hom_kernel_quotient_mul_projection(f, a, b * c)
    ring_hom_kernel_quotient_mul(f,
        quotient_over_mk(kernel_quotient_relation(f.hom), a),
        quotient_over_mk(kernel_quotient_relation(f.hom), b * c)) =
        quotient_over_mk(kernel_quotient_relation(f.hom), a * (b * c))
    (a * b) * c = a * (b * c)
    quotient_over_mk(kernel_quotient_relation(f.hom), (a * b) * c) =
        quotient_over_mk(kernel_quotient_relation(f.hom), a * (b * c))
    ring_hom_kernel_quotient_mul(f,
        ring_hom_kernel_quotient_mul(f,
            quotient_over_mk(kernel_quotient_relation(f.hom), a),
            quotient_over_mk(kernel_quotient_relation(f.hom), b)),
        quotient_over_mk(kernel_quotient_relation(f.hom), c)) =
        quotient_over_mk(kernel_quotient_relation(f.hom), (a * b) * c)
    ring_hom_kernel_quotient_mul(f,
        quotient_over_mk(kernel_quotient_relation(f.hom), a),
        ring_hom_kernel_quotient_mul(f,
            quotient_over_mk(kernel_quotient_relation(f.hom), b),
            quotient_over_mk(kernel_quotient_relation(f.hom), c))) =
        quotient_over_mk(kernel_quotient_relation(f.hom), a * (b * c))
}

/// The quotient one is a left identity for ring quotient multiplication.
theorem ring_hom_kernel_quotient_one_mul[R: Ring, S: Ring](f: RingHom[R, S], a: R) {
    ring_hom_kernel_quotient_mul(f,
        quotient_over_mk(kernel_quotient_relation(f.hom), R.1),
        quotient_over_mk(kernel_quotient_relation(f.hom), a)) =
        quotient_over_mk(kernel_quotient_relation(f.hom), a)
} by {
    ring_hom_kernel_quotient_mul_projection(f, R.1, a)
    R.1 * a = a
}

/// The quotient one is a right identity for ring quotient multiplication.
theorem ring_hom_kernel_quotient_mul_one[R: Ring, S: Ring](f: RingHom[R, S], a: R) {
    ring_hom_kernel_quotient_mul(f,
        quotient_over_mk(kernel_quotient_relation(f.hom), a),
        quotient_over_mk(kernel_quotient_relation(f.hom), R.1)) =
        quotient_over_mk(kernel_quotient_relation(f.hom), a)
} by {
    ring_hom_kernel_quotient_mul_projection(f, a, R.1)
    a * R.1 = a
}

/// Multiplication on the ring kernel quotient distributes over addition from the left.
theorem ring_hom_kernel_quotient_distrib_left[R: Ring, S: Ring](f: RingHom[R, S], a: R, b: R, c: R) {
    ring_hom_kernel_quotient_mul(f,
        quotient_over_mk(kernel_quotient_relation(f.hom), a),
        ring_hom_kernel_quotient_add(f,
            quotient_over_mk(kernel_quotient_relation(f.hom), b),
            quotient_over_mk(kernel_quotient_relation(f.hom), c))) =
    ring_hom_kernel_quotient_add(f,
        ring_hom_kernel_quotient_mul(f,
            quotient_over_mk(kernel_quotient_relation(f.hom), a),
            quotient_over_mk(kernel_quotient_relation(f.hom), b)),
        ring_hom_kernel_quotient_mul(f,
            quotient_over_mk(kernel_quotient_relation(f.hom), a),
            quotient_over_mk(kernel_quotient_relation(f.hom), c)))
} by {
    ring_hom_kernel_quotient_add_projection(f, b, c)
    ring_hom_kernel_quotient_mul(f,
        quotient_over_mk(kernel_quotient_relation(f.hom), a),
        ring_hom_kernel_quotient_add(f,
            quotient_over_mk(kernel_quotient_relation(f.hom), b),
            quotient_over_mk(kernel_quotient_relation(f.hom), c))) =
    ring_hom_kernel_quotient_mul(f,
        quotient_over_mk(kernel_quotient_relation(f.hom), a),
        quotient_over_mk(kernel_quotient_relation(f.hom), b + c))
    ring_hom_kernel_quotient_mul_projection(f, a, b + c)
    ring_hom_kernel_quotient_mul(f,
        quotient_over_mk(kernel_quotient_relation(f.hom), a),
        quotient_over_mk(kernel_quotient_relation(f.hom), b + c)) =
        quotient_over_mk(kernel_quotient_relation(f.hom), a * (b + c))
    ring_hom_kernel_quotient_mul_projection(f, a, b)
    ring_hom_kernel_quotient_mul_projection(f, a, c)
    ring_hom_kernel_quotient_add_projection(f, a * b, a * c)
    ring_hom_kernel_quotient_add(f,
        ring_hom_kernel_quotient_mul(f,
            quotient_over_mk(kernel_quotient_relation(f.hom), a),
            quotient_over_mk(kernel_quotient_relation(f.hom), b)),
        ring_hom_kernel_quotient_mul(f,
            quotient_over_mk(kernel_quotient_relation(f.hom), a),
            quotient_over_mk(kernel_quotient_relation(f.hom), c))) =
    ring_hom_kernel_quotient_add(f,
        quotient_over_mk(kernel_quotient_relation(f.hom), a * b),
        quotient_over_mk(kernel_quotient_relation(f.hom), a * c))
    ring_hom_kernel_quotient_add(f,
        quotient_over_mk(kernel_quotient_relation(f.hom), a * b),
        quotient_over_mk(kernel_quotient_relation(f.hom), a * c)) =
        quotient_over_mk(kernel_quotient_relation(f.hom), a * b + a * c)
    a * (b + c) = a * b + a * c
    quotient_over_mk(kernel_quotient_relation(f.hom), a * (b + c)) =
        quotient_over_mk(kernel_quotient_relation(f.hom), a * b + a * c)
}

/// Multiplication on the ring kernel quotient distributes over addition from the right.
theorem ring_hom_kernel_quotient_distrib_right[R: Ring, S: Ring](f: RingHom[R, S], a: R, b: R, c: R) {
    ring_hom_kernel_quotient_mul(f,
        ring_hom_kernel_quotient_add(f,
            quotient_over_mk(kernel_quotient_relation(f.hom), a),
            quotient_over_mk(kernel_quotient_relation(f.hom), b)),
        quotient_over_mk(kernel_quotient_relation(f.hom), c)) =
    ring_hom_kernel_quotient_add(f,
        ring_hom_kernel_quotient_mul(f,
            quotient_over_mk(kernel_quotient_relation(f.hom), a),
            quotient_over_mk(kernel_quotient_relation(f.hom), c)),
        ring_hom_kernel_quotient_mul(f,
            quotient_over_mk(kernel_quotient_relation(f.hom), b),
            quotient_over_mk(kernel_quotient_relation(f.hom), c)))
} by {
    ring_hom_kernel_quotient_add_projection(f, a, b)
    ring_hom_kernel_quotient_mul(f,
        ring_hom_kernel_quotient_add(f,
            quotient_over_mk(kernel_quotient_relation(f.hom), a),
            quotient_over_mk(kernel_quotient_relation(f.hom), b)),
        quotient_over_mk(kernel_quotient_relation(f.hom), c)) =
    ring_hom_kernel_quotient_mul(f,
        quotient_over_mk(kernel_quotient_relation(f.hom), a + b),
        quotient_over_mk(kernel_quotient_relation(f.hom), c))
    ring_hom_kernel_quotient_mul_projection(f, a + b, c)
    ring_hom_kernel_quotient_mul(f,
        quotient_over_mk(kernel_quotient_relation(f.hom), a + b),
        quotient_over_mk(kernel_quotient_relation(f.hom), c)) =
        quotient_over_mk(kernel_quotient_relation(f.hom), (a + b) * c)
    ring_hom_kernel_quotient_mul_projection(f, a, c)
    ring_hom_kernel_quotient_mul_projection(f, b, c)
    ring_hom_kernel_quotient_add_projection(f, a * c, b * c)
    ring_hom_kernel_quotient_add(f,
        ring_hom_kernel_quotient_mul(f,
            quotient_over_mk(kernel_quotient_relation(f.hom), a),
            quotient_over_mk(kernel_quotient_relation(f.hom), c)),
        ring_hom_kernel_quotient_mul(f,
            quotient_over_mk(kernel_quotient_relation(f.hom), b),
            quotient_over_mk(kernel_quotient_relation(f.hom), c))) =
    ring_hom_kernel_quotient_add(f,
        quotient_over_mk(kernel_quotient_relation(f.hom), a * c),
        quotient_over_mk(kernel_quotient_relation(f.hom), b * c))
    ring_hom_kernel_quotient_add(f,
        quotient_over_mk(kernel_quotient_relation(f.hom), a * c),
        quotient_over_mk(kernel_quotient_relation(f.hom), b * c)) =
        quotient_over_mk(kernel_quotient_relation(f.hom), a * c + b * c)
    (a + b) * c = a * c + b * c
    quotient_over_mk(kernel_quotient_relation(f.hom), (a + b) * c) =
        quotient_over_mk(kernel_quotient_relation(f.hom), a * c + b * c)
}

/// Multiplication on the ring-kernel quotient distributes over subtraction from the left.
theorem ring_hom_kernel_quotient_mul_sub[R: Ring, S: Ring](f: RingHom[R, S], a: R, b: R, c: R) {
    ring_hom_kernel_quotient_mul(f,
        quotient_over_mk(kernel_quotient_relation(f.hom), a),
        ring_hom_kernel_quotient_sub(f,
            quotient_over_mk(kernel_quotient_relation(f.hom), b),
            quotient_over_mk(kernel_quotient_relation(f.hom), c))) =
    ring_hom_kernel_quotient_sub(f,
        ring_hom_kernel_quotient_mul(f,
            quotient_over_mk(kernel_quotient_relation(f.hom), a),
            quotient_over_mk(kernel_quotient_relation(f.hom), b)),
        ring_hom_kernel_quotient_mul(f,
            quotient_over_mk(kernel_quotient_relation(f.hom), a),
            quotient_over_mk(kernel_quotient_relation(f.hom), c)))
} by {
    ring_hom_kernel_quotient_sub_projection(f, b, c)
    ring_hom_kernel_quotient_mul_projection(f, a, b - c)
    ring_hom_kernel_quotient_mul(f,
        quotient_over_mk(kernel_quotient_relation(f.hom), a),
        ring_hom_kernel_quotient_sub(f,
            quotient_over_mk(kernel_quotient_relation(f.hom), b),
            quotient_over_mk(kernel_quotient_relation(f.hom), c))) =
        quotient_over_mk(kernel_quotient_relation(f.hom), a * (b - c))
    ring_hom_kernel_quotient_mul_projection(f, a, b)
    ring_hom_kernel_quotient_mul_projection(f, a, c)
    ring_hom_kernel_quotient_sub_projection(f, a * b, a * c)
    ring_hom_kernel_quotient_sub(f,
        ring_hom_kernel_quotient_mul(f,
            quotient_over_mk(kernel_quotient_relation(f.hom), a),
            quotient_over_mk(kernel_quotient_relation(f.hom), b)),
        ring_hom_kernel_quotient_mul(f,
            quotient_over_mk(kernel_quotient_relation(f.hom), a),
            quotient_over_mk(kernel_quotient_relation(f.hom), c))) =
        quotient_over_mk(kernel_quotient_relation(f.hom), a * b - a * c)
    a * (b - c) = a * b - a * c
    quotient_over_mk(kernel_quotient_relation(f.hom), a * (b - c)) =
        quotient_over_mk(kernel_quotient_relation(f.hom), a * b - a * c)
}

/// Multiplication on the ring-kernel quotient distributes over subtraction from the right.
theorem ring_hom_kernel_quotient_sub_mul[R: Ring, S: Ring](f: RingHom[R, S], a: R, b: R, c: R) {
    ring_hom_kernel_quotient_mul(f,
        ring_hom_kernel_quotient_sub(f,
            quotient_over_mk(kernel_quotient_relation(f.hom), a),
            quotient_over_mk(kernel_quotient_relation(f.hom), b)),
        quotient_over_mk(kernel_quotient_relation(f.hom), c)) =
    ring_hom_kernel_quotient_sub(f,
        ring_hom_kernel_quotient_mul(f,
            quotient_over_mk(kernel_quotient_relation(f.hom), a),
            quotient_over_mk(kernel_quotient_relation(f.hom), c)),
        ring_hom_kernel_quotient_mul(f,
            quotient_over_mk(kernel_quotient_relation(f.hom), b),
            quotient_over_mk(kernel_quotient_relation(f.hom), c)))
} by {
    ring_hom_kernel_quotient_sub_projection(f, a, b)
    ring_hom_kernel_quotient_mul_projection(f, a - b, c)
    ring_hom_kernel_quotient_mul(f,
        ring_hom_kernel_quotient_sub(f,
            quotient_over_mk(kernel_quotient_relation(f.hom), a),
            quotient_over_mk(kernel_quotient_relation(f.hom), b)),
        quotient_over_mk(kernel_quotient_relation(f.hom), c)) =
        quotient_over_mk(kernel_quotient_relation(f.hom), (a - b) * c)
    ring_hom_kernel_quotient_mul_projection(f, a, c)
    ring_hom_kernel_quotient_mul_projection(f, b, c)
    ring_hom_kernel_quotient_sub_projection(f, a * c, b * c)
    ring_hom_kernel_quotient_sub(f,
        ring_hom_kernel_quotient_mul(f,
            quotient_over_mk(kernel_quotient_relation(f.hom), a),
            quotient_over_mk(kernel_quotient_relation(f.hom), c)),
        ring_hom_kernel_quotient_mul(f,
            quotient_over_mk(kernel_quotient_relation(f.hom), b),
            quotient_over_mk(kernel_quotient_relation(f.hom), c))) =
        quotient_over_mk(kernel_quotient_relation(f.hom), a * c - b * c)
    (a - b) * c = a * c - b * c
    quotient_over_mk(kernel_quotient_relation(f.hom), (a - b) * c) =
        quotient_over_mk(kernel_quotient_relation(f.hom), a * c - b * c)
}

/// Multiplying by quotient zero on the right yields quotient zero.
theorem ring_hom_kernel_quotient_mul_zero_left[R: Ring, S: Ring](f: RingHom[R, S], a: R) {
    ring_hom_kernel_quotient_mul(f,
        quotient_over_mk(kernel_quotient_relation(f.hom), a),
        quotient_over_mk(kernel_quotient_relation(f.hom), R.0)) =
        quotient_over_mk(kernel_quotient_relation(f.hom), R.0)
} by {
    ring_hom_kernel_quotient_mul_projection(f, a, R.0)
    a * R.0 = R.0
}

/// Multiplying quotient zero by anything on the right yields quotient zero.
theorem ring_hom_kernel_quotient_mul_zero_right[R: Ring, S: Ring](f: RingHom[R, S], a: R) {
    ring_hom_kernel_quotient_mul(f,
        quotient_over_mk(kernel_quotient_relation(f.hom), R.0),
        quotient_over_mk(kernel_quotient_relation(f.hom), a)) =
        quotient_over_mk(kernel_quotient_relation(f.hom), R.0)
} by {
    ring_hom_kernel_quotient_mul_projection(f, R.0, a)
    R.0 * a = R.0
}

/// The kernel quotient relation of a monoid homomorphism respects multiplication.
theorem monoid_hom_kernel_quotient_relation_respects_mul[M: Monoid, N: Monoid](f: MonoidHom[M, N]) {
    quotient_binary_respects(kernel_quotient_relation(f.hom), M.mul)
} by {
    quotient_binary_respects_eq_respects_binary_op(kernel_quotient_relation(f.hom), M.mul)
    monoid_hom_kernel_respects_mul(f)
    respects_mul_eq_respects_binary_op(kernel_relation(f.hom))
    respects_binary_op(kernel_relation(f.hom), M.mul)
    kernel_quotient_relation_rel(f.hom)
    kernel_quotient_relation(f.hom).rel = kernel_relation(f.hom)
    respects_binary_op(kernel_quotient_relation(f.hom).rel, M.mul)
}

/// Multiplication on the kernel quotient by a monoid homomorphism.
define monoid_hom_kernel_quotient_mul[M: Monoid, N: Monoid](f: MonoidHom[M, N]) ->
    (QuotientOver[M], QuotientOver[M]) -> QuotientOver[M] {
    quotient_over_binary_op(M.mul)
}

/// The identity element on the kernel quotient by a monoid homomorphism.
define monoid_hom_kernel_quotient_one[M: Monoid, N: Monoid](f: MonoidHom[M, N]) -> QuotientOver[M] {
    quotient_over_const(kernel_quotient_relation(f.hom), M.1)
}

/// The identity element on the kernel quotient is the projection of the identity.
theorem monoid_hom_kernel_quotient_one_projection[M: Monoid, N: Monoid](f: MonoidHom[M, N]) {
    monoid_hom_kernel_quotient_one(f) = quotient_over_mk(kernel_quotient_relation(f.hom), M.1)
} by {
    quotient_over_const_projection(kernel_quotient_relation(f.hom), M.1)
}

/// Multiplication on the kernel quotient agrees with multiplication of representatives.
theorem monoid_hom_kernel_quotient_mul_projection[M: Monoid, N: Monoid](f: MonoidHom[M, N], a: M, b: M) {
    monoid_hom_kernel_quotient_mul(f,
        quotient_over_mk(kernel_quotient_relation(f.hom), a),
        quotient_over_mk(kernel_quotient_relation(f.hom), b)
    ) = quotient_over_mk(kernel_quotient_relation(f.hom), a * b)
} by {
    monoid_hom_kernel_quotient_relation_respects_mul(f)
    quotient_over_binary_op_projection(kernel_quotient_relation(f.hom), M.mul, a, b)
    M.mul(a, b) = a * b
}

/// Multiplication in the kernel quotient respects related left representatives.
theorem monoid_hom_kernel_quotient_mul_projection_left[M: Monoid, N: Monoid](f: MonoidHom[M, N],
        a1: M, a2: M, b: M) {
    kernel_relation(f.hom, a1, a2) implies
    monoid_hom_kernel_quotient_mul(f,
        quotient_over_mk(kernel_quotient_relation(f.hom), a1),
        quotient_over_mk(kernel_quotient_relation(f.hom), b)
    ) = monoid_hom_kernel_quotient_mul(f,
        quotient_over_mk(kernel_quotient_relation(f.hom), a2),
        quotient_over_mk(kernel_quotient_relation(f.hom), b)
    )
} by {
    if kernel_relation(f.hom, a1, a2) {
        kernel_quotient_relation_rel(f.hom)
        kernel_quotient_relation(f.hom).rel(a1, a2)
        monoid_hom_kernel_quotient_relation_respects_mul(f)
        quotient_over_binary_op_projection_left(kernel_quotient_relation(f.hom), M.mul, a1, a2, b)
    }
}

/// Multiplication in the kernel quotient respects related right representatives.
theorem monoid_hom_kernel_quotient_mul_projection_right[M: Monoid, N: Monoid](f: MonoidHom[M, N],
        a: M, b1: M, b2: M) {
    kernel_relation(f.hom, b1, b2) implies
    monoid_hom_kernel_quotient_mul(f,
        quotient_over_mk(kernel_quotient_relation(f.hom), a),
        quotient_over_mk(kernel_quotient_relation(f.hom), b1)
    ) = monoid_hom_kernel_quotient_mul(f,
        quotient_over_mk(kernel_quotient_relation(f.hom), a),
        quotient_over_mk(kernel_quotient_relation(f.hom), b2)
    )
} by {
    if kernel_relation(f.hom, b1, b2) {
        kernel_quotient_relation_rel(f.hom)
        kernel_quotient_relation(f.hom).rel(b1, b2)
        monoid_hom_kernel_quotient_relation_respects_mul(f)
        quotient_over_binary_op_projection_right(kernel_quotient_relation(f.hom), M.mul, a, b1, b2)
    }
}

/// The identity is a left identity for multiplication on projected kernel quotient elements.
theorem monoid_hom_kernel_quotient_mul_one_left_projection[M: Monoid, N: Monoid](f: MonoidHom[M, N], a: M) {
    monoid_hom_kernel_quotient_mul(f,
        monoid_hom_kernel_quotient_one(f),
        quotient_over_mk(kernel_quotient_relation(f.hom), a)
    ) = quotient_over_mk(kernel_quotient_relation(f.hom), a)
} by {
    monoid_hom_kernel_quotient_relation_respects_mul(f)
    M.1 * a = a
    M.mul(M.1, a) = a
    quotient_over_binary_op_projection_identity_left(kernel_quotient_relation(f.hom), M.mul, M.1, a)
}

/// The identity is a right identity for multiplication on projected kernel quotient elements.
theorem monoid_hom_kernel_quotient_mul_one_right_projection[M: Monoid, N: Monoid](f: MonoidHom[M, N], a: M) {
    monoid_hom_kernel_quotient_mul(f,
        quotient_over_mk(kernel_quotient_relation(f.hom), a),
        monoid_hom_kernel_quotient_one(f)
    ) = quotient_over_mk(kernel_quotient_relation(f.hom), a)
} by {
    monoid_hom_kernel_quotient_relation_respects_mul(f)
    a * M.1 = a
    M.mul(a, M.1) = a
    quotient_over_binary_op_projection_identity_right(kernel_quotient_relation(f.hom), M.mul, M.1, a)
}

/// Powers on the kernel quotient by a monoid homomorphism.
define monoid_hom_kernel_quotient_pow[M: Monoid, N: Monoid](
    f: MonoidHom[M, N], q: QuotientOver[M], n: Nat
) -> QuotientOver[M] {
    match n {
        Nat.zero {
            monoid_hom_kernel_quotient_one(f)
        }
        Nat.suc(k) {
            monoid_hom_kernel_quotient_mul(f, q, monoid_hom_kernel_quotient_pow(f, q, k))
        }
    }
}

/// The zeroth power on the monoid kernel quotient is the quotient identity.
theorem monoid_hom_kernel_quotient_pow_zero_projection[M: Monoid, N: Monoid](
    f: MonoidHom[M, N], q: QuotientOver[M]
) {
    monoid_hom_kernel_quotient_pow(f, q, Nat.0) = monoid_hom_kernel_quotient_one(f)
} by {
}

/// The successor power on the monoid kernel quotient multiplies by the base.
theorem monoid_hom_kernel_quotient_pow_suc_projection[M: Monoid, N: Monoid](
    f: MonoidHom[M, N], q: QuotientOver[M], n: Nat
) {
    monoid_hom_kernel_quotient_pow(f, q, n.suc) =
        monoid_hom_kernel_quotient_mul(f, q, monoid_hom_kernel_quotient_pow(f, q, n))
}

/// The zeroth power on a projected representative is the projected identity.
theorem monoid_hom_kernel_quotient_pow_mk_zero[M: Monoid, N: Monoid](f: MonoidHom[M, N], a: M) {
    monoid_hom_kernel_quotient_pow(f, quotient_over_mk(kernel_quotient_relation(f.hom), a), Nat.0) =
        quotient_over_mk(kernel_quotient_relation(f.hom), a.pow(Nat.0))
} by {
    monoid_hom_kernel_quotient_pow_zero_projection(f, quotient_over_mk(kernel_quotient_relation(f.hom), a))
    monoid_hom_kernel_quotient_one_projection(f)
    a.pow(Nat.0) = M.1
}

/// Powers of projected monoid representatives agree with projected powers.
theorem monoid_hom_kernel_quotient_pow_mk[M: Monoid, N: Monoid](f: MonoidHom[M, N], a: M, n: Nat) {
    monoid_hom_kernel_quotient_pow(f, quotient_over_mk(kernel_quotient_relation(f.hom), a), n) =
        quotient_over_mk(kernel_quotient_relation(f.hom), a.pow(n))
} by {
    define p(k: Nat) -> Bool {
        monoid_hom_kernel_quotient_pow(f, quotient_over_mk(kernel_quotient_relation(f.hom), a), k) =
            quotient_over_mk(kernel_quotient_relation(f.hom), a.pow(k))
    }
    monoid_hom_kernel_quotient_pow_mk_zero(f, a)
    p(Nat.0)
    forall(k: Nat) {
        if p(k) {
            monoid_hom_kernel_quotient_pow_suc_projection(
                f, quotient_over_mk(kernel_quotient_relation(f.hom), a), k)
            monoid_hom_kernel_quotient_pow(f,
                quotient_over_mk(kernel_quotient_relation(f.hom), a), k.suc) =
                monoid_hom_kernel_quotient_mul(f,
                    quotient_over_mk(kernel_quotient_relation(f.hom), a),
                    monoid_hom_kernel_quotient_pow(f,
                        quotient_over_mk(kernel_quotient_relation(f.hom), a), k))
            monoid_hom_kernel_quotient_pow(f,
                quotient_over_mk(kernel_quotient_relation(f.hom), a), k) =
                quotient_over_mk(kernel_quotient_relation(f.hom), a.pow(k))
            monoid_hom_kernel_quotient_mul(f,
                quotient_over_mk(kernel_quotient_relation(f.hom), a),
                monoid_hom_kernel_quotient_pow(f,
                    quotient_over_mk(kernel_quotient_relation(f.hom), a), k)) =
                monoid_hom_kernel_quotient_mul(f,
                    quotient_over_mk(kernel_quotient_relation(f.hom), a),
                    quotient_over_mk(kernel_quotient_relation(f.hom), a.pow(k)))
            monoid_hom_kernel_quotient_mul_projection(f, a, a.pow(k))
            monoid_hom_kernel_quotient_mul(f,
                quotient_over_mk(kernel_quotient_relation(f.hom), a),
                quotient_over_mk(kernel_quotient_relation(f.hom), a.pow(k))) =
                quotient_over_mk(kernel_quotient_relation(f.hom), a * a.pow(k))
            a.pow(k.suc) = a * a.pow(k)
            quotient_over_mk(kernel_quotient_relation(f.hom), a * a.pow(k)) =
                quotient_over_mk(kernel_quotient_relation(f.hom), a.pow(k.suc))
            p(k.suc)
        }
    }
    p(n)
}

/// Multiplying two projected monoid-kernel powers adds their exponents.
theorem monoid_hom_kernel_quotient_pow_add[M: Monoid, N: Monoid](
    f: MonoidHom[M, N], a: M, n: Nat, m: Nat
) {
    monoid_hom_kernel_quotient_mul(f,
        monoid_hom_kernel_quotient_pow(f, quotient_over_mk(kernel_quotient_relation(f.hom), a), n),
        monoid_hom_kernel_quotient_pow(f, quotient_over_mk(kernel_quotient_relation(f.hom), a), m)) =
    monoid_hom_kernel_quotient_pow(f, quotient_over_mk(kernel_quotient_relation(f.hom), a), n + m)
} by {
    monoid_hom_kernel_quotient_pow_mk(f, a, n)
    monoid_hom_kernel_quotient_pow_mk(f, a, m)
    monoid_hom_kernel_quotient_mul(f,
        monoid_hom_kernel_quotient_pow(f, quotient_over_mk(kernel_quotient_relation(f.hom), a), n),
        monoid_hom_kernel_quotient_pow(f, quotient_over_mk(kernel_quotient_relation(f.hom), a), m)) =
    monoid_hom_kernel_quotient_mul(f,
        quotient_over_mk(kernel_quotient_relation(f.hom), a.pow(n)),
        quotient_over_mk(kernel_quotient_relation(f.hom), a.pow(m)))
    monoid_hom_kernel_quotient_mul_projection(f, a.pow(n), a.pow(m))
    monoid_hom_kernel_quotient_mul(f,
        quotient_over_mk(kernel_quotient_relation(f.hom), a.pow(n)),
        quotient_over_mk(kernel_quotient_relation(f.hom), a.pow(m))) =
        quotient_over_mk(kernel_quotient_relation(f.hom), a.pow(n) * a.pow(m))
    monoid_hom_kernel_quotient_pow_mk(f, a, n + m)
    monoid_hom_kernel_quotient_pow(f, quotient_over_mk(kernel_quotient_relation(f.hom), a), n + m) =
        quotient_over_mk(kernel_quotient_relation(f.hom), a.pow(n + m))
    pow_add(a, n, m)
    a.pow(n) * a.pow(m) = a.pow(n + m)
    quotient_over_mk(kernel_quotient_relation(f.hom), a.pow(n) * a.pow(m)) =
        quotient_over_mk(kernel_quotient_relation(f.hom), a.pow(n + m))
}

/// Raising a projected monoid-kernel power to a power multiplies the exponents.
theorem monoid_hom_kernel_quotient_pow_pow[M: Monoid, N: Monoid](
    f: MonoidHom[M, N], a: M, n: Nat, m: Nat
) {
    monoid_hom_kernel_quotient_pow(f,
        monoid_hom_kernel_quotient_pow(f, quotient_over_mk(kernel_quotient_relation(f.hom), a), n),
        m) =
    monoid_hom_kernel_quotient_pow(f, quotient_over_mk(kernel_quotient_relation(f.hom), a), n * m)
} by {
    monoid_hom_kernel_quotient_pow_mk(f, a, n)
    monoid_hom_kernel_quotient_pow(f, quotient_over_mk(kernel_quotient_relation(f.hom), a), n) =
        quotient_over_mk(kernel_quotient_relation(f.hom), a.pow(n))
    monoid_hom_kernel_quotient_pow(f,
        monoid_hom_kernel_quotient_pow(f, quotient_over_mk(kernel_quotient_relation(f.hom), a), n),
        m) =
    monoid_hom_kernel_quotient_pow(f,
        quotient_over_mk(kernel_quotient_relation(f.hom), a.pow(n)),
        m)
    monoid_hom_kernel_quotient_pow_mk(f, a.pow(n), m)
    monoid_hom_kernel_quotient_pow(f,
        quotient_over_mk(kernel_quotient_relation(f.hom), a.pow(n)),
        m) =
        quotient_over_mk(kernel_quotient_relation(f.hom), a.pow(n).pow(m))
    monoid_hom_kernel_quotient_pow_mk(f, a, n * m)
    monoid_hom_kernel_quotient_pow(f, quotient_over_mk(kernel_quotient_relation(f.hom), a), n * m) =
        quotient_over_mk(kernel_quotient_relation(f.hom), a.pow(n * m))
    pow_pow(a, n, m)
    a.pow(n).pow(m) = a.pow(n * m)
    quotient_over_mk(kernel_quotient_relation(f.hom), a.pow(n).pow(m)) =
        quotient_over_mk(kernel_quotient_relation(f.hom), a.pow(n * m))
}

/// Quotient one raised to a natural power is quotient one in a monoid-kernel quotient.
theorem monoid_hom_kernel_quotient_one_pow[M: Monoid, N: Monoid](f: MonoidHom[M, N], n: Nat) {
    monoid_hom_kernel_quotient_pow(f, monoid_hom_kernel_quotient_one(f), n) =
        monoid_hom_kernel_quotient_one(f)
} by {
    monoid_hom_kernel_quotient_one_projection(f)
    monoid_hom_kernel_quotient_pow(f, monoid_hom_kernel_quotient_one(f), n) =
    monoid_hom_kernel_quotient_pow(f, quotient_over_mk(kernel_quotient_relation(f.hom), M.1), n)
    monoid_hom_kernel_quotient_pow_mk(f, M.1, n)
    monoid_hom_kernel_quotient_pow(f, quotient_over_mk(kernel_quotient_relation(f.hom), M.1), n) =
        quotient_over_mk(kernel_quotient_relation(f.hom), M.1.pow(n))
    one_pow[M](n)
    M.1.pow(n) = M.1
    quotient_over_mk(kernel_quotient_relation(f.hom), M.1.pow(n)) =
        quotient_over_mk(kernel_quotient_relation(f.hom), M.1)
}

/// The kernel quotient relation of an additive monoid homomorphism respects addition.
theorem add_monoid_hom_kernel_quotient_relation_respects_add[A: AddMonoid, B: AddMonoid](f: AddMonoidHom[A, B]) {
    quotient_binary_respects(kernel_quotient_relation(f.hom), A.add)
} by {
    quotient_binary_respects_eq_respects_binary_op(kernel_quotient_relation(f.hom), A.add)
    is_add_monoid_hom(f.hom)
    add_monoid_hom_kernel_respects_add(f.hom)
    respects_add_eq_respects_binary_op(kernel_relation(f.hom))
    respects_binary_op(kernel_relation(f.hom), A.add)
    kernel_quotient_relation_rel(f.hom)
    kernel_quotient_relation(f.hom).rel = kernel_relation(f.hom)
    respects_binary_op(kernel_quotient_relation(f.hom).rel, A.add)
}

/// Addition on the kernel quotient by an additive monoid homomorphism.
define add_monoid_hom_kernel_quotient_add[A: AddMonoid, B: AddMonoid](f: AddMonoidHom[A, B]) ->
    (QuotientOver[A], QuotientOver[A]) -> QuotientOver[A] {
    quotient_over_binary_op(A.add)
}

/// The zero element on the kernel quotient by an additive monoid homomorphism.
define add_monoid_hom_kernel_quotient_zero[A: AddMonoid, B: AddMonoid](f: AddMonoidHom[A, B]) -> QuotientOver[A] {
    quotient_over_const(kernel_quotient_relation(f.hom), A.0)
}

/// The zero element on the kernel quotient is the projection of zero.
theorem add_monoid_hom_kernel_quotient_zero_projection[A: AddMonoid, B: AddMonoid](f: AddMonoidHom[A, B]) {
    add_monoid_hom_kernel_quotient_zero(f) = quotient_over_mk(kernel_quotient_relation(f.hom), A.0)
} by {
    quotient_over_const_projection(kernel_quotient_relation(f.hom), A.0)
}

/// Addition on the kernel quotient agrees with addition of representatives.
theorem add_monoid_hom_kernel_quotient_add_projection[A: AddMonoid, B: AddMonoid](f: AddMonoidHom[A, B], a: A, b: A) {
    add_monoid_hom_kernel_quotient_add(f,
        quotient_over_mk(kernel_quotient_relation(f.hom), a),
        quotient_over_mk(kernel_quotient_relation(f.hom), b)
    ) = quotient_over_mk(kernel_quotient_relation(f.hom), a + b)
} by {
    add_monoid_hom_kernel_quotient_relation_respects_add(f)
    quotient_over_binary_op_projection(kernel_quotient_relation(f.hom), A.add, a, b)
    A.add(a, b) = a + b
}

/// Addition in the kernel quotient respects related left representatives.
theorem add_monoid_hom_kernel_quotient_add_projection_left[A: AddMonoid, B: AddMonoid](f: AddMonoidHom[A, B],
        a1: A, a2: A, b: A) {
    kernel_relation(f.hom, a1, a2) implies
    add_monoid_hom_kernel_quotient_add(f,
        quotient_over_mk(kernel_quotient_relation(f.hom), a1),
        quotient_over_mk(kernel_quotient_relation(f.hom), b)
    ) = add_monoid_hom_kernel_quotient_add(f,
        quotient_over_mk(kernel_quotient_relation(f.hom), a2),
        quotient_over_mk(kernel_quotient_relation(f.hom), b)
    )
} by {
    if kernel_relation(f.hom, a1, a2) {
        kernel_quotient_relation_rel(f.hom)
        kernel_quotient_relation(f.hom).rel(a1, a2)
        add_monoid_hom_kernel_quotient_relation_respects_add(f)
        quotient_over_binary_op_projection_left(kernel_quotient_relation(f.hom), A.add, a1, a2, b)
    }
}

/// Addition in the kernel quotient respects related right representatives.
theorem add_monoid_hom_kernel_quotient_add_projection_right[A: AddMonoid, B: AddMonoid](f: AddMonoidHom[A, B],
        a: A, b1: A, b2: A) {
    kernel_relation(f.hom, b1, b2) implies
    add_monoid_hom_kernel_quotient_add(f,
        quotient_over_mk(kernel_quotient_relation(f.hom), a),
        quotient_over_mk(kernel_quotient_relation(f.hom), b1)
    ) = add_monoid_hom_kernel_quotient_add(f,
        quotient_over_mk(kernel_quotient_relation(f.hom), a),
        quotient_over_mk(kernel_quotient_relation(f.hom), b2)
    )
} by {
    if kernel_relation(f.hom, b1, b2) {
        kernel_quotient_relation_rel(f.hom)
        kernel_quotient_relation(f.hom).rel(b1, b2)
        add_monoid_hom_kernel_quotient_relation_respects_add(f)
        quotient_over_binary_op_projection_right(kernel_quotient_relation(f.hom), A.add, a, b1, b2)
    }
}

/// Zero is a left identity for addition on projected kernel quotient elements.
theorem add_monoid_hom_kernel_quotient_add_zero_left_projection[A: AddMonoid, B: AddMonoid](
    f: AddMonoidHom[A, B], a: A
) {
    add_monoid_hom_kernel_quotient_add(f,
        add_monoid_hom_kernel_quotient_zero(f),
        quotient_over_mk(kernel_quotient_relation(f.hom), a)
    ) = quotient_over_mk(kernel_quotient_relation(f.hom), a)
} by {
    add_monoid_hom_kernel_quotient_relation_respects_add(f)
    A.0 + a = a
    A.add(A.0, a) = a
    quotient_over_binary_op_projection_identity_left(kernel_quotient_relation(f.hom), A.add, A.0, a)
}

/// Zero is a right identity for addition on projected kernel quotient elements.
theorem add_monoid_hom_kernel_quotient_add_zero_right_projection[A: AddMonoid, B: AddMonoid](
    f: AddMonoidHom[A, B], a: A
) {
    add_monoid_hom_kernel_quotient_add(f,
        quotient_over_mk(kernel_quotient_relation(f.hom), a),
        add_monoid_hom_kernel_quotient_zero(f)
    ) = quotient_over_mk(kernel_quotient_relation(f.hom), a)
} by {
    add_monoid_hom_kernel_quotient_relation_respects_add(f)
    a + A.0 = a
    A.add(a, A.0) = a
    quotient_over_binary_op_projection_identity_right(kernel_quotient_relation(f.hom), A.add, A.0, a)
}

/// Multiplication on the monoid kernel quotient is associative.
theorem monoid_hom_kernel_quotient_mul_assoc[M: Monoid, N: Monoid](f: MonoidHom[M, N], a: M, b: M, c: M) {
    monoid_hom_kernel_quotient_mul(f,
        monoid_hom_kernel_quotient_mul(f,
            quotient_over_mk(kernel_quotient_relation(f.hom), a),
            quotient_over_mk(kernel_quotient_relation(f.hom), b)),
        quotient_over_mk(kernel_quotient_relation(f.hom), c)) =
    monoid_hom_kernel_quotient_mul(f,
        quotient_over_mk(kernel_quotient_relation(f.hom), a),
        monoid_hom_kernel_quotient_mul(f,
            quotient_over_mk(kernel_quotient_relation(f.hom), b),
            quotient_over_mk(kernel_quotient_relation(f.hom), c)))
} by {
    monoid_hom_kernel_quotient_mul_projection(f, a, b)
    monoid_hom_kernel_quotient_mul(f,
        quotient_over_mk(kernel_quotient_relation(f.hom), a),
        quotient_over_mk(kernel_quotient_relation(f.hom), b)) =
        quotient_over_mk(kernel_quotient_relation(f.hom), a * b)
    monoid_hom_kernel_quotient_mul(f,
        monoid_hom_kernel_quotient_mul(f,
            quotient_over_mk(kernel_quotient_relation(f.hom), a),
            quotient_over_mk(kernel_quotient_relation(f.hom), b)),
        quotient_over_mk(kernel_quotient_relation(f.hom), c)) =
    monoid_hom_kernel_quotient_mul(f,
        quotient_over_mk(kernel_quotient_relation(f.hom), a * b),
        quotient_over_mk(kernel_quotient_relation(f.hom), c))
    monoid_hom_kernel_quotient_mul_projection(f, a * b, c)
    monoid_hom_kernel_quotient_mul(f,
        quotient_over_mk(kernel_quotient_relation(f.hom), a * b),
        quotient_over_mk(kernel_quotient_relation(f.hom), c)) =
        quotient_over_mk(kernel_quotient_relation(f.hom), (a * b) * c)
    monoid_hom_kernel_quotient_mul_projection(f, b, c)
    monoid_hom_kernel_quotient_mul(f,
        quotient_over_mk(kernel_quotient_relation(f.hom), b),
        quotient_over_mk(kernel_quotient_relation(f.hom), c)) =
        quotient_over_mk(kernel_quotient_relation(f.hom), b * c)
    monoid_hom_kernel_quotient_mul(f,
        quotient_over_mk(kernel_quotient_relation(f.hom), a),
        monoid_hom_kernel_quotient_mul(f,
            quotient_over_mk(kernel_quotient_relation(f.hom), b),
            quotient_over_mk(kernel_quotient_relation(f.hom), c))) =
    monoid_hom_kernel_quotient_mul(f,
        quotient_over_mk(kernel_quotient_relation(f.hom), a),
        quotient_over_mk(kernel_quotient_relation(f.hom), b * c))
    monoid_hom_kernel_quotient_mul_projection(f, a, b * c)
    monoid_hom_kernel_quotient_mul(f,
        quotient_over_mk(kernel_quotient_relation(f.hom), a),
        quotient_over_mk(kernel_quotient_relation(f.hom), b * c)) =
        quotient_over_mk(kernel_quotient_relation(f.hom), a * (b * c))
    (a * b) * c = a * (b * c)
    quotient_over_mk(kernel_quotient_relation(f.hom), (a * b) * c) =
        quotient_over_mk(kernel_quotient_relation(f.hom), a * (b * c))
    monoid_hom_kernel_quotient_mul(f,
        monoid_hom_kernel_quotient_mul(f,
            quotient_over_mk(kernel_quotient_relation(f.hom), a),
            quotient_over_mk(kernel_quotient_relation(f.hom), b)),
        quotient_over_mk(kernel_quotient_relation(f.hom), c)) =
        quotient_over_mk(kernel_quotient_relation(f.hom), (a * b) * c)
    monoid_hom_kernel_quotient_mul(f,
        quotient_over_mk(kernel_quotient_relation(f.hom), a),
        monoid_hom_kernel_quotient_mul(f,
            quotient_over_mk(kernel_quotient_relation(f.hom), b),
            quotient_over_mk(kernel_quotient_relation(f.hom), c))) =
        quotient_over_mk(kernel_quotient_relation(f.hom), a * (b * c))
}

/// The quotient one is a left identity for monoid quotient multiplication.
theorem monoid_hom_kernel_quotient_one_mul[M: Monoid, N: Monoid](f: MonoidHom[M, N], a: M) {
    monoid_hom_kernel_quotient_mul(f,
        quotient_over_mk(kernel_quotient_relation(f.hom), M.1),
        quotient_over_mk(kernel_quotient_relation(f.hom), a)) =
        quotient_over_mk(kernel_quotient_relation(f.hom), a)
} by {
    monoid_hom_kernel_quotient_mul_projection(f, M.1, a)
    M.1 * a = a
}

/// The quotient one is a right identity for monoid quotient multiplication.
theorem monoid_hom_kernel_quotient_mul_one[M: Monoid, N: Monoid](f: MonoidHom[M, N], a: M) {
    monoid_hom_kernel_quotient_mul(f,
        quotient_over_mk(kernel_quotient_relation(f.hom), a),
        quotient_over_mk(kernel_quotient_relation(f.hom), M.1)) =
        quotient_over_mk(kernel_quotient_relation(f.hom), a)
} by {
    monoid_hom_kernel_quotient_mul_projection(f, a, M.1)
    a * M.1 = a
}

/// Multiplication on the commutative-monoid kernel quotient is commutative.
theorem monoid_hom_kernel_quotient_mul_comm[M: CommMonoid, N: CommMonoid](
    f: MonoidHom[M, N], a: M, b: M
) {
    monoid_hom_kernel_quotient_mul(f,
        quotient_over_mk(kernel_quotient_relation(f.hom), a),
        quotient_over_mk(kernel_quotient_relation(f.hom), b)) =
    monoid_hom_kernel_quotient_mul(f,
        quotient_over_mk(kernel_quotient_relation(f.hom), b),
        quotient_over_mk(kernel_quotient_relation(f.hom), a))
} by {
    monoid_hom_kernel_quotient_mul_projection(f, a, b)
    monoid_hom_kernel_quotient_mul(f,
        quotient_over_mk(kernel_quotient_relation(f.hom), a),
        quotient_over_mk(kernel_quotient_relation(f.hom), b)) =
        quotient_over_mk(kernel_quotient_relation(f.hom), a * b)
    monoid_hom_kernel_quotient_mul_projection(f, b, a)
    monoid_hom_kernel_quotient_mul(f,
        quotient_over_mk(kernel_quotient_relation(f.hom), b),
        quotient_over_mk(kernel_quotient_relation(f.hom), a)) =
        quotient_over_mk(kernel_quotient_relation(f.hom), b * a)
    a * b = b * a
    quotient_over_mk(kernel_quotient_relation(f.hom), a * b) =
        quotient_over_mk(kernel_quotient_relation(f.hom), b * a)
    monoid_hom_kernel_quotient_mul(f,
        quotient_over_mk(kernel_quotient_relation(f.hom), a),
        quotient_over_mk(kernel_quotient_relation(f.hom), b)) =
    monoid_hom_kernel_quotient_mul(f,
        quotient_over_mk(kernel_quotient_relation(f.hom), b),
        quotient_over_mk(kernel_quotient_relation(f.hom), a))
}

/// Addition on the additive-monoid kernel quotient is associative.
theorem add_monoid_hom_kernel_quotient_add_assoc[A: AddMonoid, B: AddMonoid](f: AddMonoidHom[A, B], a: A, b: A, c: A) {
    add_monoid_hom_kernel_quotient_add(f,
        add_monoid_hom_kernel_quotient_add(f,
            quotient_over_mk(kernel_quotient_relation(f.hom), a),
            quotient_over_mk(kernel_quotient_relation(f.hom), b)),
        quotient_over_mk(kernel_quotient_relation(f.hom), c)) =
    add_monoid_hom_kernel_quotient_add(f,
        quotient_over_mk(kernel_quotient_relation(f.hom), a),
        add_monoid_hom_kernel_quotient_add(f,
            quotient_over_mk(kernel_quotient_relation(f.hom), b),
            quotient_over_mk(kernel_quotient_relation(f.hom), c)))
} by {
    add_monoid_hom_kernel_quotient_add_projection(f, a, b)
    add_monoid_hom_kernel_quotient_add(f,
        quotient_over_mk(kernel_quotient_relation(f.hom), a),
        quotient_over_mk(kernel_quotient_relation(f.hom), b)) =
        quotient_over_mk(kernel_quotient_relation(f.hom), a + b)
    add_monoid_hom_kernel_quotient_add(f,
        add_monoid_hom_kernel_quotient_add(f,
            quotient_over_mk(kernel_quotient_relation(f.hom), a),
            quotient_over_mk(kernel_quotient_relation(f.hom), b)),
        quotient_over_mk(kernel_quotient_relation(f.hom), c)) =
    add_monoid_hom_kernel_quotient_add(f,
        quotient_over_mk(kernel_quotient_relation(f.hom), a + b),
        quotient_over_mk(kernel_quotient_relation(f.hom), c))
    add_monoid_hom_kernel_quotient_add_projection(f, a + b, c)
    add_monoid_hom_kernel_quotient_add(f,
        quotient_over_mk(kernel_quotient_relation(f.hom), a + b),
        quotient_over_mk(kernel_quotient_relation(f.hom), c)) =
        quotient_over_mk(kernel_quotient_relation(f.hom), (a + b) + c)
    add_monoid_hom_kernel_quotient_add_projection(f, b, c)
    add_monoid_hom_kernel_quotient_add(f,
        quotient_over_mk(kernel_quotient_relation(f.hom), b),
        quotient_over_mk(kernel_quotient_relation(f.hom), c)) =
        quotient_over_mk(kernel_quotient_relation(f.hom), b + c)
    add_monoid_hom_kernel_quotient_add(f,
        quotient_over_mk(kernel_quotient_relation(f.hom), a),
        add_monoid_hom_kernel_quotient_add(f,
            quotient_over_mk(kernel_quotient_relation(f.hom), b),
            quotient_over_mk(kernel_quotient_relation(f.hom), c))) =
    add_monoid_hom_kernel_quotient_add(f,
        quotient_over_mk(kernel_quotient_relation(f.hom), a),
        quotient_over_mk(kernel_quotient_relation(f.hom), b + c))
    add_monoid_hom_kernel_quotient_add_projection(f, a, b + c)
    add_monoid_hom_kernel_quotient_add(f,
        quotient_over_mk(kernel_quotient_relation(f.hom), a),
        quotient_over_mk(kernel_quotient_relation(f.hom), b + c)) =
        quotient_over_mk(kernel_quotient_relation(f.hom), a + (b + c))
    (a + b) + c = a + (b + c)
    quotient_over_mk(kernel_quotient_relation(f.hom), (a + b) + c) =
        quotient_over_mk(kernel_quotient_relation(f.hom), a + (b + c))
    add_monoid_hom_kernel_quotient_add(f,
        add_monoid_hom_kernel_quotient_add(f,
            quotient_over_mk(kernel_quotient_relation(f.hom), a),
            quotient_over_mk(kernel_quotient_relation(f.hom), b)),
        quotient_over_mk(kernel_quotient_relation(f.hom), c)) =
        quotient_over_mk(kernel_quotient_relation(f.hom), (a + b) + c)
    add_monoid_hom_kernel_quotient_add(f,
        quotient_over_mk(kernel_quotient_relation(f.hom), a),
        add_monoid_hom_kernel_quotient_add(f,
            quotient_over_mk(kernel_quotient_relation(f.hom), b),
            quotient_over_mk(kernel_quotient_relation(f.hom), c))) =
        quotient_over_mk(kernel_quotient_relation(f.hom), a + (b + c))
}

/// The quotient zero is a left identity for additive-monoid quotient addition.
theorem add_monoid_hom_kernel_quotient_zero_add[A: AddMonoid, B: AddMonoid](f: AddMonoidHom[A, B], a: A) {
    add_monoid_hom_kernel_quotient_add(f,
        quotient_over_mk(kernel_quotient_relation(f.hom), A.0),
        quotient_over_mk(kernel_quotient_relation(f.hom), a)) =
        quotient_over_mk(kernel_quotient_relation(f.hom), a)
} by {
    add_monoid_hom_kernel_quotient_add_projection(f, A.0, a)
    A.0 + a = a
}

/// The quotient zero is a right identity for additive-monoid quotient addition.
theorem add_monoid_hom_kernel_quotient_add_zero[A: AddMonoid, B: AddMonoid](f: AddMonoidHom[A, B], a: A) {
    add_monoid_hom_kernel_quotient_add(f,
        quotient_over_mk(kernel_quotient_relation(f.hom), a),
        quotient_over_mk(kernel_quotient_relation(f.hom), A.0)) =
        quotient_over_mk(kernel_quotient_relation(f.hom), a)
} by {
    add_monoid_hom_kernel_quotient_add_projection(f, a, A.0)
    a + A.0 = a
}

/// Addition on the additive-commutative-monoid kernel quotient is commutative.
theorem add_monoid_hom_kernel_quotient_add_comm[A: AddCommMonoid, B: AddCommMonoid](
    f: AddMonoidHom[A, B], a: A, b: A
) {
    add_monoid_hom_kernel_quotient_add(f,
        quotient_over_mk(kernel_quotient_relation(f.hom), a),
        quotient_over_mk(kernel_quotient_relation(f.hom), b)) =
    add_monoid_hom_kernel_quotient_add(f,
        quotient_over_mk(kernel_quotient_relation(f.hom), b),
        quotient_over_mk(kernel_quotient_relation(f.hom), a))
} by {
    add_monoid_hom_kernel_quotient_add_projection(f, a, b)
    add_monoid_hom_kernel_quotient_add(f,
        quotient_over_mk(kernel_quotient_relation(f.hom), a),
        quotient_over_mk(kernel_quotient_relation(f.hom), b)) =
        quotient_over_mk(kernel_quotient_relation(f.hom), a + b)
    add_monoid_hom_kernel_quotient_add_projection(f, b, a)
    add_monoid_hom_kernel_quotient_add(f,
        quotient_over_mk(kernel_quotient_relation(f.hom), b),
        quotient_over_mk(kernel_quotient_relation(f.hom), a)) =
        quotient_over_mk(kernel_quotient_relation(f.hom), b + a)
    a + b = b + a
    quotient_over_mk(kernel_quotient_relation(f.hom), a + b) =
        quotient_over_mk(kernel_quotient_relation(f.hom), b + a)
    add_monoid_hom_kernel_quotient_add(f,
        quotient_over_mk(kernel_quotient_relation(f.hom), a),
        quotient_over_mk(kernel_quotient_relation(f.hom), b)) =
    add_monoid_hom_kernel_quotient_add(f,
        quotient_over_mk(kernel_quotient_relation(f.hom), b),
        quotient_over_mk(kernel_quotient_relation(f.hom), a))
}

/// The kernel quotient relation of a linear map respects addition.
theorem linear_map_kernel_quotient_relation_respects_add[R: Ring, M: AddCommGroup, N: AddCommGroup](
    src: Module[R, M], dst: Module[R, N], f: M -> N
) {
    is_linear_map(src, dst, f) implies quotient_binary_respects(kernel_quotient_relation(f), M.add)
} by {
    if is_linear_map(src, dst, f) {
        quotient_binary_respects_eq_respects_binary_op(kernel_quotient_relation(f), M.add)
        linear_map_kernel_respects_add(src, dst, f)
        respects_add_eq_respects_binary_op(kernel_relation(f))
        respects_binary_op(kernel_relation(f), M.add)
        kernel_quotient_relation_rel(f)
        kernel_quotient_relation(f).rel = kernel_relation(f)
        respects_binary_op(kernel_quotient_relation(f).rel, M.add)
    }
}

/// The kernel of a linear map is preserved under scalar multiplication by a fixed scalar.
theorem linear_map_kernel_respects_smul[R: Ring, M: AddCommGroup, N: AddCommGroup](
    src: Module[R, M], dst: Module[R, N], f: M -> N, r: R
) {
    is_linear_map(src, dst, f) implies respects_unary_op(kernel_relation(f), src.smul(r))
} by {
    if is_linear_map(src, dst, f) {
        forall(x: M, y: M) {
            if kernel_relation(f, x, y) {
                linear_map_kernel_smul_step(src, dst, f, r, x, y)
                kernel_relation(f, src.smul(r, x), src.smul(r, y))
                kernel_relation(f, src.smul(r)(x), src.smul(r)(y))
            }
        }
    }
}

/// The kernel quotient relation of a linear map respects scalar multiplication by a fixed scalar.
theorem linear_map_kernel_quotient_relation_respects_smul[R: Ring, M: AddCommGroup, N: AddCommGroup](
    src: Module[R, M], dst: Module[R, N], f: M -> N, r: R
) {
    is_linear_map(src, dst, f) implies quotient_unary_respects(kernel_quotient_relation(f), src.smul(r))
} by {
    if is_linear_map(src, dst, f) {
        quotient_unary_respects_eq_respects_unary_op(kernel_quotient_relation(f), src.smul(r))
        linear_map_kernel_respects_smul(src, dst, f, r)
        kernel_quotient_relation_rel(f)
        kernel_quotient_relation(f).rel = kernel_relation(f)
        respects_unary_op(kernel_quotient_relation(f).rel, src.smul(r))
    }
}

/// Addition on the kernel quotient by a linear map.
define linear_map_kernel_quotient_add[R: Ring, M: AddCommGroup, N: AddCommGroup](
    src: Module[R, M], dst: Module[R, N], f: M -> N
) -> (QuotientOver[M], QuotientOver[M]) -> QuotientOver[M] {
    quotient_over_binary_op(M.add)
}

/// The zero element on the kernel quotient by a linear map.
define linear_map_kernel_quotient_zero[R: Ring, M: AddCommGroup, N: AddCommGroup](
    src: Module[R, M], dst: Module[R, N], f: M -> N
) -> QuotientOver[M] {
    quotient_over_const(kernel_quotient_relation(f), M.0)
}

/// The zero element on the kernel quotient is the projection of zero.
theorem linear_map_kernel_quotient_zero_projection[R: Ring, M: AddCommGroup, N: AddCommGroup](
    src: Module[R, M], dst: Module[R, N], f: M -> N
) {
    linear_map_kernel_quotient_zero(src, dst, f) = quotient_over_mk(kernel_quotient_relation(f), M.0)
} by {
    quotient_over_const_projection(kernel_quotient_relation(f), M.0)
}

/// Addition on the kernel quotient agrees with addition of representatives.
theorem linear_map_kernel_quotient_add_projection[R: Ring, M: AddCommGroup, N: AddCommGroup](
    src: Module[R, M], dst: Module[R, N], f: M -> N, a: M, b: M
) {
    is_linear_map(src, dst, f) implies
    linear_map_kernel_quotient_add(src, dst, f,
        quotient_over_mk(kernel_quotient_relation(f), a),
        quotient_over_mk(kernel_quotient_relation(f), b)
    ) = quotient_over_mk(kernel_quotient_relation(f), a + b)
} by {
    if is_linear_map(src, dst, f) {
        linear_map_kernel_quotient_relation_respects_add(src, dst, f)
        quotient_over_binary_op_projection(kernel_quotient_relation(f), M.add, a, b)
        M.add(a, b) = a + b
    }
}

/// Addition in the kernel quotient respects related left representatives.
theorem linear_map_kernel_quotient_add_projection_left[R: Ring, M: AddCommGroup, N: AddCommGroup](
    src: Module[R, M], dst: Module[R, N], f: M -> N, a1: M, a2: M, b: M
) {
    is_linear_map(src, dst, f) and kernel_relation(f, a1, a2) implies
    linear_map_kernel_quotient_add(src, dst, f,
        quotient_over_mk(kernel_quotient_relation(f), a1),
        quotient_over_mk(kernel_quotient_relation(f), b)
    ) = linear_map_kernel_quotient_add(src, dst, f,
        quotient_over_mk(kernel_quotient_relation(f), a2),
        quotient_over_mk(kernel_quotient_relation(f), b)
    )
} by {
    if is_linear_map(src, dst, f) and kernel_relation(f, a1, a2) {
        kernel_quotient_relation_rel(f)
        kernel_quotient_relation(f).rel(a1, a2)
        linear_map_kernel_quotient_relation_respects_add(src, dst, f)
        quotient_over_binary_op_projection_left(kernel_quotient_relation(f), M.add, a1, a2, b)
    }
}

/// Addition in the kernel quotient respects related right representatives.
theorem linear_map_kernel_quotient_add_projection_right[R: Ring, M: AddCommGroup, N: AddCommGroup](
    src: Module[R, M], dst: Module[R, N], f: M -> N, a: M, b1: M, b2: M
) {
    is_linear_map(src, dst, f) and kernel_relation(f, b1, b2) implies
    linear_map_kernel_quotient_add(src, dst, f,
        quotient_over_mk(kernel_quotient_relation(f), a),
        quotient_over_mk(kernel_quotient_relation(f), b1)
    ) = linear_map_kernel_quotient_add(src, dst, f,
        quotient_over_mk(kernel_quotient_relation(f), a),
        quotient_over_mk(kernel_quotient_relation(f), b2)
    )
} by {
    if is_linear_map(src, dst, f) and kernel_relation(f, b1, b2) {
        kernel_quotient_relation_rel(f)
        kernel_quotient_relation(f).rel(b1, b2)
        linear_map_kernel_quotient_relation_respects_add(src, dst, f)
        quotient_over_binary_op_projection_right(kernel_quotient_relation(f), M.add, a, b1, b2)
    }
}

/// Zero is a left identity for addition on projected linear-map kernel quotient elements.
theorem linear_map_kernel_quotient_add_zero_left_projection[R: Ring, M: AddCommGroup, N: AddCommGroup](
    src: Module[R, M], dst: Module[R, N], f: M -> N, a: M
) {
    is_linear_map(src, dst, f) implies
    linear_map_kernel_quotient_add(src, dst, f,
        linear_map_kernel_quotient_zero(src, dst, f),
        quotient_over_mk(kernel_quotient_relation(f), a)
    ) = quotient_over_mk(kernel_quotient_relation(f), a)
} by {
    if is_linear_map(src, dst, f) {
        linear_map_kernel_quotient_relation_respects_add(src, dst, f)
        M.0 + a = a
        M.add(M.0, a) = a
        quotient_over_binary_op_projection_identity_left(kernel_quotient_relation(f), M.add, M.0, a)
    }
}

/// Zero is a right identity for addition on projected linear-map kernel quotient elements.
theorem linear_map_kernel_quotient_add_zero_right_projection[R: Ring, M: AddCommGroup, N: AddCommGroup](
    src: Module[R, M], dst: Module[R, N], f: M -> N, a: M
) {
    is_linear_map(src, dst, f) implies
    linear_map_kernel_quotient_add(src, dst, f,
        quotient_over_mk(kernel_quotient_relation(f), a),
        linear_map_kernel_quotient_zero(src, dst, f)
    ) = quotient_over_mk(kernel_quotient_relation(f), a)
} by {
    if is_linear_map(src, dst, f) {
        linear_map_kernel_quotient_relation_respects_add(src, dst, f)
        a + M.0 = a
        M.add(a, M.0) = a
        quotient_over_binary_op_projection_identity_right(kernel_quotient_relation(f), M.add, M.0, a)
    }
}

/// Scalar multiplication by a fixed scalar on the kernel quotient by a linear map.
define linear_map_kernel_quotient_smul[R: Ring, M: AddCommGroup, N: AddCommGroup](
    src: Module[R, M], dst: Module[R, N], f: M -> N, r: R
) -> QuotientOver[M] -> QuotientOver[M] {
    quotient_over_unary_op(kernel_quotient_relation(f), src.smul(r))
}

/// Scalar multiplication on the kernel quotient agrees with scalar multiplication of representatives.
theorem linear_map_kernel_quotient_smul_projection[R: Ring, M: AddCommGroup, N: AddCommGroup](
    src: Module[R, M], dst: Module[R, N], f: M -> N, r: R, a: M
) {
    is_linear_map(src, dst, f) implies
    linear_map_kernel_quotient_smul(src, dst, f, r,
        quotient_over_mk(kernel_quotient_relation(f), a)
    ) = quotient_over_mk(kernel_quotient_relation(f), src.smul(r, a))
} by {
    if is_linear_map(src, dst, f) {
        linear_map_kernel_quotient_relation_respects_smul(src, dst, f, r)
        quotient_over_unary_op_projection(kernel_quotient_relation(f), src.smul(r), a)
        src.smul(r)(a) = src.smul(r, a)
    }
}

/// Scalar multiplication on the kernel quotient respects related representatives.
theorem linear_map_kernel_quotient_smul_projection_compatible[R: Ring, M: AddCommGroup, N: AddCommGroup](
    src: Module[R, M], dst: Module[R, N], f: M -> N, r: R, a: M, b: M
) {
    is_linear_map(src, dst, f) and kernel_relation(f, a, b) implies
    linear_map_kernel_quotient_smul(src, dst, f, r,
        quotient_over_mk(kernel_quotient_relation(f), a)
    ) = linear_map_kernel_quotient_smul(src, dst, f, r,
        quotient_over_mk(kernel_quotient_relation(f), b)
    )
} by {
    if is_linear_map(src, dst, f) and kernel_relation(f, a, b) {
        kernel_quotient_relation_rel(f)
        kernel_quotient_relation(f).rel(a, b)
        linear_map_kernel_quotient_relation_respects_smul(src, dst, f, r)
        quotient_over_unary_op_projection_compatible(kernel_quotient_relation(f), src.smul(r), a, b)
    }
}

/// Addition on the module kernel quotient is associative.
theorem linear_map_kernel_quotient_add_assoc[R: Ring, M: AddCommGroup, N: AddCommGroup](
    src: Module[R, M], dst: Module[R, N], f: M -> N, a: M, b: M, c: M
) {
    is_linear_map(src, dst, f) implies
    linear_map_kernel_quotient_add(src, dst, f,
        linear_map_kernel_quotient_add(src, dst, f,
            quotient_over_mk(kernel_quotient_relation(f), a),
            quotient_over_mk(kernel_quotient_relation(f), b)),
        quotient_over_mk(kernel_quotient_relation(f), c)) =
    linear_map_kernel_quotient_add(src, dst, f,
        quotient_over_mk(kernel_quotient_relation(f), a),
        linear_map_kernel_quotient_add(src, dst, f,
            quotient_over_mk(kernel_quotient_relation(f), b),
            quotient_over_mk(kernel_quotient_relation(f), c)))
} by {
    if is_linear_map(src, dst, f) {
        linear_map_kernel_quotient_add_projection(src, dst, f, a, b)
        linear_map_kernel_quotient_add(src, dst, f,
            quotient_over_mk(kernel_quotient_relation(f), a),
            quotient_over_mk(kernel_quotient_relation(f), b)) =
            quotient_over_mk(kernel_quotient_relation(f), a + b)
        linear_map_kernel_quotient_add(src, dst, f,
            linear_map_kernel_quotient_add(src, dst, f,
                quotient_over_mk(kernel_quotient_relation(f), a),
                quotient_over_mk(kernel_quotient_relation(f), b)),
            quotient_over_mk(kernel_quotient_relation(f), c)) =
        linear_map_kernel_quotient_add(src, dst, f,
            quotient_over_mk(kernel_quotient_relation(f), a + b),
            quotient_over_mk(kernel_quotient_relation(f), c))
        linear_map_kernel_quotient_add_projection(src, dst, f, a + b, c)
        linear_map_kernel_quotient_add(src, dst, f,
            quotient_over_mk(kernel_quotient_relation(f), a + b),
            quotient_over_mk(kernel_quotient_relation(f), c)) =
            quotient_over_mk(kernel_quotient_relation(f), (a + b) + c)
        linear_map_kernel_quotient_add_projection(src, dst, f, b, c)
        linear_map_kernel_quotient_add(src, dst, f,
            quotient_over_mk(kernel_quotient_relation(f), b),
            quotient_over_mk(kernel_quotient_relation(f), c)) =
            quotient_over_mk(kernel_quotient_relation(f), b + c)
        linear_map_kernel_quotient_add(src, dst, f,
            quotient_over_mk(kernel_quotient_relation(f), a),
            linear_map_kernel_quotient_add(src, dst, f,
                quotient_over_mk(kernel_quotient_relation(f), b),
                quotient_over_mk(kernel_quotient_relation(f), c))) =
        linear_map_kernel_quotient_add(src, dst, f,
            quotient_over_mk(kernel_quotient_relation(f), a),
            quotient_over_mk(kernel_quotient_relation(f), b + c))
        linear_map_kernel_quotient_add_projection(src, dst, f, a, b + c)
        linear_map_kernel_quotient_add(src, dst, f,
            quotient_over_mk(kernel_quotient_relation(f), a),
            quotient_over_mk(kernel_quotient_relation(f), b + c)) =
            quotient_over_mk(kernel_quotient_relation(f), a + (b + c))
        (a + b) + c = a + (b + c)
        quotient_over_mk(kernel_quotient_relation(f), (a + b) + c) =
            quotient_over_mk(kernel_quotient_relation(f), a + (b + c))
        linear_map_kernel_quotient_add(src, dst, f,
            linear_map_kernel_quotient_add(src, dst, f,
                quotient_over_mk(kernel_quotient_relation(f), a),
                quotient_over_mk(kernel_quotient_relation(f), b)),
            quotient_over_mk(kernel_quotient_relation(f), c)) =
            quotient_over_mk(kernel_quotient_relation(f), (a + b) + c)
        linear_map_kernel_quotient_add(src, dst, f,
            quotient_over_mk(kernel_quotient_relation(f), a),
            linear_map_kernel_quotient_add(src, dst, f,
                quotient_over_mk(kernel_quotient_relation(f), b),
                quotient_over_mk(kernel_quotient_relation(f), c))) =
            quotient_over_mk(kernel_quotient_relation(f), a + (b + c))
        linear_map_kernel_quotient_add(src, dst, f,
            linear_map_kernel_quotient_add(src, dst, f,
                quotient_over_mk(kernel_quotient_relation(f), a),
                quotient_over_mk(kernel_quotient_relation(f), b)),
            quotient_over_mk(kernel_quotient_relation(f), c)) =
        linear_map_kernel_quotient_add(src, dst, f,
            quotient_over_mk(kernel_quotient_relation(f), a),
            linear_map_kernel_quotient_add(src, dst, f,
                quotient_over_mk(kernel_quotient_relation(f), b),
                quotient_over_mk(kernel_quotient_relation(f), c)))
    }
}

/// The quotient zero is a left identity for module quotient addition.
theorem linear_map_kernel_quotient_zero_add[R: Ring, M: AddCommGroup, N: AddCommGroup](
    src: Module[R, M], dst: Module[R, N], f: M -> N, a: M
) {
    is_linear_map(src, dst, f) implies
    linear_map_kernel_quotient_add(src, dst, f,
        quotient_over_mk(kernel_quotient_relation(f), M.0),
        quotient_over_mk(kernel_quotient_relation(f), a)) =
        quotient_over_mk(kernel_quotient_relation(f), a)
} by {
    if is_linear_map(src, dst, f) {
        linear_map_kernel_quotient_add_projection(src, dst, f, M.0, a)
        M.0 + a = a
    }
}

/// The quotient zero is a right identity for module quotient addition.
theorem linear_map_kernel_quotient_add_zero[R: Ring, M: AddCommGroup, N: AddCommGroup](
    src: Module[R, M], dst: Module[R, N], f: M -> N, a: M
) {
    is_linear_map(src, dst, f) implies
    linear_map_kernel_quotient_add(src, dst, f,
        quotient_over_mk(kernel_quotient_relation(f), a),
        quotient_over_mk(kernel_quotient_relation(f), M.0)) =
        quotient_over_mk(kernel_quotient_relation(f), a)
} by {
    if is_linear_map(src, dst, f) {
        linear_map_kernel_quotient_add_projection(src, dst, f, a, M.0)
        a + M.0 = a
    }
}

/// Addition on the module kernel quotient is commutative.
theorem linear_map_kernel_quotient_add_comm[R: Ring, M: AddCommGroup, N: AddCommGroup](
    src: Module[R, M], dst: Module[R, N], f: M -> N, a: M, b: M
) {
    is_linear_map(src, dst, f) implies
    linear_map_kernel_quotient_add(src, dst, f,
        quotient_over_mk(kernel_quotient_relation(f), a),
        quotient_over_mk(kernel_quotient_relation(f), b)) =
    linear_map_kernel_quotient_add(src, dst, f,
        quotient_over_mk(kernel_quotient_relation(f), b),
        quotient_over_mk(kernel_quotient_relation(f), a))
} by {
    if is_linear_map(src, dst, f) {
        linear_map_kernel_quotient_add_projection(src, dst, f, a, b)
        linear_map_kernel_quotient_add(src, dst, f,
            quotient_over_mk(kernel_quotient_relation(f), a),
            quotient_over_mk(kernel_quotient_relation(f), b)) =
            quotient_over_mk(kernel_quotient_relation(f), a + b)
        linear_map_kernel_quotient_add_projection(src, dst, f, b, a)
        linear_map_kernel_quotient_add(src, dst, f,
            quotient_over_mk(kernel_quotient_relation(f), b),
            quotient_over_mk(kernel_quotient_relation(f), a)) =
            quotient_over_mk(kernel_quotient_relation(f), b + a)
        a + b = b + a
        quotient_over_mk(kernel_quotient_relation(f), a + b) =
            quotient_over_mk(kernel_quotient_relation(f), b + a)
        linear_map_kernel_quotient_add(src, dst, f,
            quotient_over_mk(kernel_quotient_relation(f), a),
            quotient_over_mk(kernel_quotient_relation(f), b)) =
        linear_map_kernel_quotient_add(src, dst, f,
            quotient_over_mk(kernel_quotient_relation(f), b),
            quotient_over_mk(kernel_quotient_relation(f), a))
    }
}

/// The ring identity acts as the identity scalar on the module kernel quotient.
theorem linear_map_kernel_quotient_smul_one[R: Ring, M: AddCommGroup, N: AddCommGroup](
    src: Module[R, M], dst: Module[R, N], f: M -> N, a: M
) {
    is_linear_map(src, dst, f) implies
    linear_map_kernel_quotient_smul(src, dst, f, R.1,
        quotient_over_mk(kernel_quotient_relation(f), a)) =
        quotient_over_mk(kernel_quotient_relation(f), a)
} by {
    if is_linear_map(src, dst, f) {
        linear_map_kernel_quotient_smul_projection(src, dst, f, R.1, a)
        linear_map_kernel_quotient_smul(src, dst, f, R.1,
            quotient_over_mk(kernel_quotient_relation(f), a)) =
            quotient_over_mk(kernel_quotient_relation(f), src.smul(R.1, a))
        module_smul_one(src, a)
        src.smul(R.1, a) = a
        quotient_over_mk(kernel_quotient_relation(f), src.smul(R.1, a)) =
            quotient_over_mk(kernel_quotient_relation(f), a)
        linear_map_kernel_quotient_smul(src, dst, f, R.1,
            quotient_over_mk(kernel_quotient_relation(f), a)) =
            quotient_over_mk(kernel_quotient_relation(f), a)
    }
}

/// Scalar multiplication on the module kernel quotient is associative with ring multiplication.
theorem linear_map_kernel_quotient_smul_assoc[R: Ring, M: AddCommGroup, N: AddCommGroup](
    src: Module[R, M], dst: Module[R, N], f: M -> N, r: R, s: R, a: M
) {
    is_linear_map(src, dst, f) implies
    linear_map_kernel_quotient_smul(src, dst, f, r * s,
        quotient_over_mk(kernel_quotient_relation(f), a)) =
    linear_map_kernel_quotient_smul(src, dst, f, r,
        linear_map_kernel_quotient_smul(src, dst, f, s,
            quotient_over_mk(kernel_quotient_relation(f), a)))
} by {
    if is_linear_map(src, dst, f) {
        linear_map_kernel_quotient_smul_projection(src, dst, f, r * s, a)
        linear_map_kernel_quotient_smul(src, dst, f, r * s,
            quotient_over_mk(kernel_quotient_relation(f), a)) =
            quotient_over_mk(kernel_quotient_relation(f), src.smul(r * s, a))
        linear_map_kernel_quotient_smul_projection(src, dst, f, s, a)
        linear_map_kernel_quotient_smul(src, dst, f, s,
            quotient_over_mk(kernel_quotient_relation(f), a)) =
            quotient_over_mk(kernel_quotient_relation(f), src.smul(s, a))
        linear_map_kernel_quotient_smul_projection(src, dst, f, r, src.smul(s, a))
        linear_map_kernel_quotient_smul(src, dst, f, r,
            quotient_over_mk(kernel_quotient_relation(f), src.smul(s, a))) =
            quotient_over_mk(kernel_quotient_relation(f), src.smul(r, src.smul(s, a)))
        linear_map_kernel_quotient_smul(src, dst, f, r,
            linear_map_kernel_quotient_smul(src, dst, f, s,
                quotient_over_mk(kernel_quotient_relation(f), a))) =
        linear_map_kernel_quotient_smul(src, dst, f, r,
            quotient_over_mk(kernel_quotient_relation(f), src.smul(s, a)))
        module_smul_assoc(src, r, s, a)
        src.smul(r * s, a) = src.smul(r, src.smul(s, a))
        quotient_over_mk(kernel_quotient_relation(f), src.smul(r * s, a)) =
            quotient_over_mk(kernel_quotient_relation(f), src.smul(r, src.smul(s, a)))
        linear_map_kernel_quotient_smul(src, dst, f, r * s,
            quotient_over_mk(kernel_quotient_relation(f), a)) =
        linear_map_kernel_quotient_smul(src, dst, f, r,
            linear_map_kernel_quotient_smul(src, dst, f, s,
                quotient_over_mk(kernel_quotient_relation(f), a)))
    }
}

/// Scalar multiplication on the module kernel quotient distributes over quotient addition.
theorem linear_map_kernel_quotient_smul_add_right[R: Ring, M: AddCommGroup, N: AddCommGroup](
    src: Module[R, M], dst: Module[R, N], f: M -> N, r: R, a: M, b: M
) {
    is_linear_map(src, dst, f) implies
    linear_map_kernel_quotient_smul(src, dst, f, r,
        linear_map_kernel_quotient_add(src, dst, f,
            quotient_over_mk(kernel_quotient_relation(f), a),
            quotient_over_mk(kernel_quotient_relation(f), b))) =
    linear_map_kernel_quotient_add(src, dst, f,
        linear_map_kernel_quotient_smul(src, dst, f, r,
            quotient_over_mk(kernel_quotient_relation(f), a)),
        linear_map_kernel_quotient_smul(src, dst, f, r,
            quotient_over_mk(kernel_quotient_relation(f), b)))
} by {
    if is_linear_map(src, dst, f) {
        linear_map_kernel_quotient_add_projection(src, dst, f, a, b)
        linear_map_kernel_quotient_add(src, dst, f,
            quotient_over_mk(kernel_quotient_relation(f), a),
            quotient_over_mk(kernel_quotient_relation(f), b)) =
            quotient_over_mk(kernel_quotient_relation(f), a + b)
        linear_map_kernel_quotient_smul(src, dst, f, r,
            linear_map_kernel_quotient_add(src, dst, f,
                quotient_over_mk(kernel_quotient_relation(f), a),
                quotient_over_mk(kernel_quotient_relation(f), b))) =
        linear_map_kernel_quotient_smul(src, dst, f, r,
            quotient_over_mk(kernel_quotient_relation(f), a + b))
        linear_map_kernel_quotient_smul_projection(src, dst, f, r, a + b)
        linear_map_kernel_quotient_smul(src, dst, f, r,
            quotient_over_mk(kernel_quotient_relation(f), a + b)) =
            quotient_over_mk(kernel_quotient_relation(f), src.smul(r, a + b))
        linear_map_kernel_quotient_smul_projection(src, dst, f, r, a)
        linear_map_kernel_quotient_smul_projection(src, dst, f, r, b)
        linear_map_kernel_quotient_add_projection(src, dst, f, src.smul(r, a), src.smul(r, b))
        linear_map_kernel_quotient_add(src, dst, f,
            linear_map_kernel_quotient_smul(src, dst, f, r,
                quotient_over_mk(kernel_quotient_relation(f), a)),
            linear_map_kernel_quotient_smul(src, dst, f, r,
                quotient_over_mk(kernel_quotient_relation(f), b))) =
        linear_map_kernel_quotient_add(src, dst, f,
            quotient_over_mk(kernel_quotient_relation(f), src.smul(r, a)),
            quotient_over_mk(kernel_quotient_relation(f), src.smul(r, b)))
        linear_map_kernel_quotient_add(src, dst, f,
            quotient_over_mk(kernel_quotient_relation(f), src.smul(r, a)),
            quotient_over_mk(kernel_quotient_relation(f), src.smul(r, b))) =
            quotient_over_mk(kernel_quotient_relation(f), src.smul(r, a) + src.smul(r, b))
        module_smul_add_right(src, r, a, b)
        src.smul(r, a + b) = src.smul(r, a) + src.smul(r, b)
        quotient_over_mk(kernel_quotient_relation(f), src.smul(r, a + b)) =
            quotient_over_mk(kernel_quotient_relation(f), src.smul(r, a) + src.smul(r, b))
        linear_map_kernel_quotient_smul(src, dst, f, r,
            linear_map_kernel_quotient_add(src, dst, f,
                quotient_over_mk(kernel_quotient_relation(f), a),
                quotient_over_mk(kernel_quotient_relation(f), b))) =
        linear_map_kernel_quotient_add(src, dst, f,
            linear_map_kernel_quotient_smul(src, dst, f, r,
                quotient_over_mk(kernel_quotient_relation(f), a)),
            linear_map_kernel_quotient_smul(src, dst, f, r,
                quotient_over_mk(kernel_quotient_relation(f), b)))
    }
}

/// Sum of scalars distributes through scalar multiplication on the module kernel quotient.
theorem linear_map_kernel_quotient_smul_add_left[R: Ring, M: AddCommGroup, N: AddCommGroup](
    src: Module[R, M], dst: Module[R, N], f: M -> N, r: R, s: R, a: M
) {
    is_linear_map(src, dst, f) implies
    linear_map_kernel_quotient_smul(src, dst, f, r + s,
        quotient_over_mk(kernel_quotient_relation(f), a)) =
    linear_map_kernel_quotient_add(src, dst, f,
        linear_map_kernel_quotient_smul(src, dst, f, r,
            quotient_over_mk(kernel_quotient_relation(f), a)),
        linear_map_kernel_quotient_smul(src, dst, f, s,
            quotient_over_mk(kernel_quotient_relation(f), a)))
} by {
    if is_linear_map(src, dst, f) {
        linear_map_kernel_quotient_smul_projection(src, dst, f, r + s, a)
        linear_map_kernel_quotient_smul(src, dst, f, r + s,
            quotient_over_mk(kernel_quotient_relation(f), a)) =
            quotient_over_mk(kernel_quotient_relation(f), src.smul(r + s, a))
        linear_map_kernel_quotient_smul_projection(src, dst, f, r, a)
        linear_map_kernel_quotient_smul_projection(src, dst, f, s, a)
        linear_map_kernel_quotient_add_projection(src, dst, f, src.smul(r, a), src.smul(s, a))
        linear_map_kernel_quotient_add(src, dst, f,
            linear_map_kernel_quotient_smul(src, dst, f, r,
                quotient_over_mk(kernel_quotient_relation(f), a)),
            linear_map_kernel_quotient_smul(src, dst, f, s,
                quotient_over_mk(kernel_quotient_relation(f), a))) =
        linear_map_kernel_quotient_add(src, dst, f,
            quotient_over_mk(kernel_quotient_relation(f), src.smul(r, a)),
            quotient_over_mk(kernel_quotient_relation(f), src.smul(s, a)))
        linear_map_kernel_quotient_add(src, dst, f,
            quotient_over_mk(kernel_quotient_relation(f), src.smul(r, a)),
            quotient_over_mk(kernel_quotient_relation(f), src.smul(s, a))) =
            quotient_over_mk(kernel_quotient_relation(f), src.smul(r, a) + src.smul(s, a))
        module_smul_add_left(src, r, s, a)
        src.smul(r + s, a) = src.smul(r, a) + src.smul(s, a)
        quotient_over_mk(kernel_quotient_relation(f), src.smul(r + s, a)) =
            quotient_over_mk(kernel_quotient_relation(f), src.smul(r, a) + src.smul(s, a))
        linear_map_kernel_quotient_smul(src, dst, f, r + s,
            quotient_over_mk(kernel_quotient_relation(f), a)) =
        linear_map_kernel_quotient_add(src, dst, f,
            linear_map_kernel_quotient_smul(src, dst, f, r,
                quotient_over_mk(kernel_quotient_relation(f), a)),
            linear_map_kernel_quotient_smul(src, dst, f, s,
                quotient_over_mk(kernel_quotient_relation(f), a)))
    }
}

/// The zero scalar acts as zero on the module kernel quotient.
theorem linear_map_kernel_quotient_smul_zero_left[R: Ring, M: AddCommGroup, N: AddCommGroup](
    src: Module[R, M], dst: Module[R, N], f: M -> N, a: M
) {
    is_linear_map(src, dst, f) implies
    linear_map_kernel_quotient_smul(src, dst, f, R.0,
        quotient_over_mk(kernel_quotient_relation(f), a)) =
        quotient_over_mk(kernel_quotient_relation(f), M.0)
} by {
    if is_linear_map(src, dst, f) {
        linear_map_kernel_quotient_smul_projection(src, dst, f, R.0, a)
        linear_map_kernel_quotient_smul(src, dst, f, R.0,
            quotient_over_mk(kernel_quotient_relation(f), a)) =
            quotient_over_mk(kernel_quotient_relation(f), src.smul(R.0, a))
        module_smul_zero_left(src, a)
        src.smul(R.0, a) = M.0
        quotient_over_mk(kernel_quotient_relation(f), src.smul(R.0, a)) =
            quotient_over_mk(kernel_quotient_relation(f), M.0)
        linear_map_kernel_quotient_smul(src, dst, f, R.0,
            quotient_over_mk(kernel_quotient_relation(f), a)) =
            quotient_over_mk(kernel_quotient_relation(f), M.0)
    }
}

/// Any scalar applied to the quotient zero gives the quotient zero.
theorem linear_map_kernel_quotient_smul_zero_right[R: Ring, M: AddCommGroup, N: AddCommGroup](
    src: Module[R, M], dst: Module[R, N], f: M -> N, r: R
) {
    is_linear_map(src, dst, f) implies
    linear_map_kernel_quotient_smul(src, dst, f, r,
        quotient_over_mk(kernel_quotient_relation(f), M.0)) =
        quotient_over_mk(kernel_quotient_relation(f), M.0)
} by {
    if is_linear_map(src, dst, f) {
        linear_map_kernel_quotient_smul_projection(src, dst, f, r, M.0)
        linear_map_kernel_quotient_smul(src, dst, f, r,
            quotient_over_mk(kernel_quotient_relation(f), M.0)) =
            quotient_over_mk(kernel_quotient_relation(f), src.smul(r, M.0))
        module_smul_zero_right(src, r)
        src.smul(r, M.0) = M.0
        quotient_over_mk(kernel_quotient_relation(f), src.smul(r, M.0)) =
            quotient_over_mk(kernel_quotient_relation(f), M.0)
        linear_map_kernel_quotient_smul(src, dst, f, r,
            quotient_over_mk(kernel_quotient_relation(f), M.0)) =
            quotient_over_mk(kernel_quotient_relation(f), M.0)
    }
}

/// Multiplication on the commutative-ring kernel quotient is commutative.
theorem ring_hom_kernel_quotient_mul_comm[R: CommRing, S: CommRing](f: RingHom[R, S], a: R, b: R) {
    ring_hom_kernel_quotient_mul(f,
        quotient_over_mk(kernel_quotient_relation(f.hom), a),
        quotient_over_mk(kernel_quotient_relation(f.hom), b)) =
    ring_hom_kernel_quotient_mul(f,
        quotient_over_mk(kernel_quotient_relation(f.hom), b),
        quotient_over_mk(kernel_quotient_relation(f.hom), a))
} by {
    ring_hom_kernel_quotient_mul_projection(f, a, b)
    ring_hom_kernel_quotient_mul(f,
        quotient_over_mk(kernel_quotient_relation(f.hom), a),
        quotient_over_mk(kernel_quotient_relation(f.hom), b)) =
        quotient_over_mk(kernel_quotient_relation(f.hom), a * b)
    ring_hom_kernel_quotient_mul_projection(f, b, a)
    ring_hom_kernel_quotient_mul(f,
        quotient_over_mk(kernel_quotient_relation(f.hom), b),
        quotient_over_mk(kernel_quotient_relation(f.hom), a)) =
        quotient_over_mk(kernel_quotient_relation(f.hom), b * a)
    a * b = b * a
    quotient_over_mk(kernel_quotient_relation(f.hom), a * b) =
        quotient_over_mk(kernel_quotient_relation(f.hom), b * a)
    ring_hom_kernel_quotient_mul(f,
        quotient_over_mk(kernel_quotient_relation(f.hom), a),
        quotient_over_mk(kernel_quotient_relation(f.hom), b)) =
    ring_hom_kernel_quotient_mul(f,
        quotient_over_mk(kernel_quotient_relation(f.hom), b),
        quotient_over_mk(kernel_quotient_relation(f.hom), a))
}

/// The kernel of a linear map is preserved under additive negation.
theorem linear_map_kernel_respects_neg[R: Ring, M: AddCommGroup, N: AddCommGroup](
    src: Module[R, M], dst: Module[R, N], f: M -> N
) {
    is_linear_map(src, dst, f) implies respects_unary_op(kernel_relation(f), M.neg)
} by {
    if is_linear_map(src, dst, f) {
        forall(a: M, b: M) {
            if kernel_relation(f, a, b) {
                f(a) = f(b)
                linear_map_neg(src, dst, f, a)
                linear_map_neg(src, dst, f, b)
                f(-a) = -f(a)
                f(-b) = -f(b)
                -f(a) = -f(b)
                f(-a) = f(-b)
                kernel_relation(f, -a, -b)
                kernel_relation(f, M.neg(a), M.neg(b))
            }
        }
    }
}

/// The kernel of a linear map is an additive-group congruence.
theorem linear_map_kernel_is_add_group_congruence[R: Ring, M: AddCommGroup, N: AddCommGroup](
    src: Module[R, M], dst: Module[R, N], f: M -> N
) {
    is_linear_map(src, dst, f) implies is_add_group_congruence(kernel_relation(f))
} by {
    if is_linear_map(src, dst, f) {
        linear_map_kernel_is_add_congruence(src, dst, f)
        kernel_relation_is_equivalence(f)
        linear_map_kernel_respects_neg(src, dst, f)
        is_unary_congruence(kernel_relation(f), M.neg)
    }
}

/// The kernel quotient relation of a linear map respects additive negation.
theorem linear_map_kernel_quotient_relation_respects_neg[R: Ring, M: AddCommGroup, N: AddCommGroup](
    src: Module[R, M], dst: Module[R, N], f: M -> N
) {
    is_linear_map(src, dst, f) implies quotient_unary_respects(kernel_quotient_relation(f), M.neg)
} by {
    if is_linear_map(src, dst, f) {
        quotient_unary_respects_eq_respects_unary_op(kernel_quotient_relation(f), M.neg)
        linear_map_kernel_respects_neg(src, dst, f)
        kernel_quotient_relation_rel(f)
        kernel_quotient_relation(f).rel = kernel_relation(f)
        respects_unary_op(kernel_quotient_relation(f).rel, M.neg)
    }
}

/// Negation on the module kernel quotient.
define linear_map_kernel_quotient_neg[R: Ring, M: AddCommGroup, N: AddCommGroup](
    src: Module[R, M], dst: Module[R, N], f: M -> N
) -> QuotientOver[M] -> QuotientOver[M] {
    quotient_over_unary_op(kernel_quotient_relation(f), M.neg)
}

/// Negation on the module kernel quotient agrees with negation of representatives.
theorem linear_map_kernel_quotient_neg_projection[R: Ring, M: AddCommGroup, N: AddCommGroup](
    src: Module[R, M], dst: Module[R, N], f: M -> N, a: M
) {
    is_linear_map(src, dst, f) implies
    linear_map_kernel_quotient_neg(src, dst, f,
        quotient_over_mk(kernel_quotient_relation(f), a)) =
        quotient_over_mk(kernel_quotient_relation(f), -a)
} by {
    if is_linear_map(src, dst, f) {
        linear_map_kernel_quotient_relation_respects_neg(src, dst, f)
        quotient_over_unary_op_projection(kernel_quotient_relation(f), M.neg, a)
        M.neg(a) = -a
    }
}

/// Negation on the module kernel quotient respects related representatives.
theorem linear_map_kernel_quotient_neg_projection_compatible[R: Ring, M: AddCommGroup, N: AddCommGroup](
    src: Module[R, M], dst: Module[R, N], f: M -> N, a: M, b: M
) {
    is_linear_map(src, dst, f) and kernel_relation(f, a, b) implies
    linear_map_kernel_quotient_neg(src, dst, f,
        quotient_over_mk(kernel_quotient_relation(f), a)) =
    linear_map_kernel_quotient_neg(src, dst, f,
        quotient_over_mk(kernel_quotient_relation(f), b))
} by {
    if is_linear_map(src, dst, f) and kernel_relation(f, a, b) {
        kernel_quotient_relation_rel(f)
        kernel_quotient_relation(f).rel(a, b)
        linear_map_kernel_quotient_relation_respects_neg(src, dst, f)
        quotient_over_unary_op_projection_compatible(kernel_quotient_relation(f), M.neg, a, b)
    }
}

/// Quotient negation is a left additive inverse on the module kernel quotient.
theorem linear_map_kernel_quotient_neg_add[R: Ring, M: AddCommGroup, N: AddCommGroup](
    src: Module[R, M], dst: Module[R, N], f: M -> N, a: M
) {
    is_linear_map(src, dst, f) implies
    linear_map_kernel_quotient_add(src, dst, f,
        linear_map_kernel_quotient_neg(src, dst, f,
            quotient_over_mk(kernel_quotient_relation(f), a)),
        quotient_over_mk(kernel_quotient_relation(f), a)) =
        quotient_over_mk(kernel_quotient_relation(f), M.0)
} by {
    if is_linear_map(src, dst, f) {
        linear_map_kernel_quotient_neg_projection(src, dst, f, a)
        linear_map_kernel_quotient_neg(src, dst, f,
            quotient_over_mk(kernel_quotient_relation(f), a)) =
            quotient_over_mk(kernel_quotient_relation(f), -a)
        linear_map_kernel_quotient_add(src, dst, f,
            linear_map_kernel_quotient_neg(src, dst, f,
                quotient_over_mk(kernel_quotient_relation(f), a)),
            quotient_over_mk(kernel_quotient_relation(f), a)) =
        linear_map_kernel_quotient_add(src, dst, f,
            quotient_over_mk(kernel_quotient_relation(f), -a),
            quotient_over_mk(kernel_quotient_relation(f), a))
        linear_map_kernel_quotient_add_projection(src, dst, f, -a, a)
        linear_map_kernel_quotient_add(src, dst, f,
            quotient_over_mk(kernel_quotient_relation(f), -a),
            quotient_over_mk(kernel_quotient_relation(f), a)) =
            quotient_over_mk(kernel_quotient_relation(f), -a + a)
        -a + a = M.0
        quotient_over_mk(kernel_quotient_relation(f), -a + a) =
            quotient_over_mk(kernel_quotient_relation(f), M.0)
        linear_map_kernel_quotient_add(src, dst, f,
            linear_map_kernel_quotient_neg(src, dst, f,
                quotient_over_mk(kernel_quotient_relation(f), a)),
            quotient_over_mk(kernel_quotient_relation(f), a)) =
            quotient_over_mk(kernel_quotient_relation(f), M.0)
    }
}

/// Quotient negation is a right additive inverse on the module kernel quotient.
theorem linear_map_kernel_quotient_add_neg[R: Ring, M: AddCommGroup, N: AddCommGroup](
    src: Module[R, M], dst: Module[R, N], f: M -> N, a: M
) {
    is_linear_map(src, dst, f) implies
    linear_map_kernel_quotient_add(src, dst, f,
        quotient_over_mk(kernel_quotient_relation(f), a),
        linear_map_kernel_quotient_neg(src, dst, f,
            quotient_over_mk(kernel_quotient_relation(f), a))) =
        quotient_over_mk(kernel_quotient_relation(f), M.0)
} by {
    if is_linear_map(src, dst, f) {
        linear_map_kernel_quotient_neg_projection(src, dst, f, a)
        linear_map_kernel_quotient_neg(src, dst, f,
            quotient_over_mk(kernel_quotient_relation(f), a)) =
            quotient_over_mk(kernel_quotient_relation(f), -a)
        linear_map_kernel_quotient_add(src, dst, f,
            quotient_over_mk(kernel_quotient_relation(f), a),
            linear_map_kernel_quotient_neg(src, dst, f,
                quotient_over_mk(kernel_quotient_relation(f), a))) =
        linear_map_kernel_quotient_add(src, dst, f,
            quotient_over_mk(kernel_quotient_relation(f), a),
            quotient_over_mk(kernel_quotient_relation(f), -a))
        linear_map_kernel_quotient_add_projection(src, dst, f, a, -a)
        linear_map_kernel_quotient_add(src, dst, f,
            quotient_over_mk(kernel_quotient_relation(f), a),
            quotient_over_mk(kernel_quotient_relation(f), -a)) =
            quotient_over_mk(kernel_quotient_relation(f), a + -a)
        a + -a = M.0
        quotient_over_mk(kernel_quotient_relation(f), a + -a) =
            quotient_over_mk(kernel_quotient_relation(f), M.0)
        linear_map_kernel_quotient_add(src, dst, f,
            quotient_over_mk(kernel_quotient_relation(f), a),
            linear_map_kernel_quotient_neg(src, dst, f,
                quotient_over_mk(kernel_quotient_relation(f), a))) =
            quotient_over_mk(kernel_quotient_relation(f), M.0)
    }
}

/// The canonical projection to a linear-map-kernel quotient.
define linear_map_kernel_quotient_project[R: Ring, M: AddCommGroup, N: AddCommGroup](
    src: Module[R, M], dst: Module[R, N], f: M -> N, a: M
) -> QuotientOver[M] {
    quotient_over_mk(kernel_quotient_relation(f), a)
}

/// The canonical linear-map-kernel projection is the quotient constructor.
theorem linear_map_kernel_quotient_project_eq_mk[R: Ring, M: AddCommGroup, N: AddCommGroup](
    src: Module[R, M], dst: Module[R, N], f: M -> N, a: M
) {
    linear_map_kernel_quotient_project(src, dst, f, a) =
        quotient_over_mk(kernel_quotient_relation(f), a)
}

/// Equality of projected linear-map-kernel representatives is the kernel relation.
theorem linear_map_kernel_quotient_project_eq_iff_kernel_relation[R: Ring, M: AddCommGroup, N: AddCommGroup](
    src: Module[R, M], dst: Module[R, N], f: M -> N, a: M, b: M
) {
    (linear_map_kernel_quotient_project(src, dst, f, a) = linear_map_kernel_quotient_project(src, dst, f, b)) =
        kernel_relation(f, a, b)
} by {
    linear_map_kernel_quotient_project_eq_mk(src, dst, f, a)
    linear_map_kernel_quotient_project_eq_mk(src, dst, f, b)
    quotient_over_mk_eq_iff_rel(kernel_quotient_relation(f), a, b)
    kernel_quotient_relation_rel(f)
}

/// The map out of the linear-map-kernel quotient induced by `f`.
define linear_map_kernel_quotient_induced[R: Ring, M: AddCommGroup, N: AddCommGroup](
    src: Module[R, M], dst: Module[R, N], f: M -> N
) -> QuotientOver[M] -> N {
    quotient_over_lift_to(f)
}

/// The induced map agrees with `f` on canonical linear-map-kernel projections.
theorem linear_map_kernel_quotient_induced_project[R: Ring, M: AddCommGroup, N: AddCommGroup](
    src: Module[R, M], dst: Module[R, N], f: M -> N, a: M
) {
    linear_map_kernel_quotient_induced(src, dst, f,
        linear_map_kernel_quotient_project(src, dst, f, a)) = f(a)
} by {
    kernel_relation_respects(f)
    linear_map_kernel_quotient_project_eq_mk(src, dst, f, a)
    quotient_over_lift_to_projection(kernel_quotient_relation(f), f, a)
}

/// The induced map out of the linear-map-kernel quotient separates projections of unrelated representatives.
theorem linear_map_kernel_quotient_induced_injective[R: Ring, M: AddCommGroup, N: AddCommGroup](
    src: Module[R, M], dst: Module[R, N], f: M -> N, a: M, b: M
) {
    linear_map_kernel_quotient_induced(src, dst, f,
        linear_map_kernel_quotient_project(src, dst, f, a)) =
        linear_map_kernel_quotient_induced(src, dst, f,
            linear_map_kernel_quotient_project(src, dst, f, b))
    implies linear_map_kernel_quotient_project(src, dst, f, a) =
        linear_map_kernel_quotient_project(src, dst, f, b)
} by {
    if linear_map_kernel_quotient_induced(src, dst, f,
        linear_map_kernel_quotient_project(src, dst, f, a)) =
        linear_map_kernel_quotient_induced(src, dst, f,
            linear_map_kernel_quotient_project(src, dst, f, b)) {
        linear_map_kernel_quotient_induced_project(src, dst, f, a)
        linear_map_kernel_quotient_induced_project(src, dst, f, b)
        f(a) = f(b)
        kernel_relation(f, a, b) = (f(a) = f(b))
        kernel_relation(f, a, b)
        linear_map_kernel_quotient_project_eq_iff_kernel_relation(src, dst, f, a, b)
        linear_map_kernel_quotient_project(src, dst, f, a) =
            linear_map_kernel_quotient_project(src, dst, f, b)
    }
}

/// The canonical linear-map-kernel projection preserves zero.
theorem linear_map_kernel_quotient_project_zero[R: Ring, M: AddCommGroup, N: AddCommGroup](
    src: Module[R, M], dst: Module[R, N], f: M -> N
) {
    linear_map_kernel_quotient_project(src, dst, f, M.0) =
        linear_map_kernel_quotient_zero(src, dst, f)
} by {
    linear_map_kernel_quotient_zero_projection(src, dst, f)
}

/// The canonical linear-map-kernel projection preserves addition.
theorem linear_map_kernel_quotient_project_add[R: Ring, M: AddCommGroup, N: AddCommGroup](
    src: Module[R, M], dst: Module[R, N], f: M -> N, a: M, b: M
) {
    is_linear_map(src, dst, f) implies
    linear_map_kernel_quotient_project(src, dst, f, a + b) =
    linear_map_kernel_quotient_add(src, dst, f,
        linear_map_kernel_quotient_project(src, dst, f, a),
        linear_map_kernel_quotient_project(src, dst, f, b))
} by {
    if is_linear_map(src, dst, f) {
        linear_map_kernel_quotient_add_projection(src, dst, f, a, b)
    }
}

/// The canonical linear-map-kernel projection preserves negation.
theorem linear_map_kernel_quotient_project_neg[R: Ring, M: AddCommGroup, N: AddCommGroup](
    src: Module[R, M], dst: Module[R, N], f: M -> N, a: M
) {
    is_linear_map(src, dst, f) implies
    linear_map_kernel_quotient_project(src, dst, f, -a) =
    linear_map_kernel_quotient_neg(src, dst, f,
        linear_map_kernel_quotient_project(src, dst, f, a))
} by {
    if is_linear_map(src, dst, f) {
        linear_map_kernel_quotient_neg_projection(src, dst, f, a)
    }
}

/// The canonical linear-map-kernel projection preserves scalar multiplication.
theorem linear_map_kernel_quotient_project_smul[R: Ring, M: AddCommGroup, N: AddCommGroup](
    src: Module[R, M], dst: Module[R, N], f: M -> N, r: R, a: M
) {
    is_linear_map(src, dst, f) implies
    linear_map_kernel_quotient_project(src, dst, f, src.smul(r, a)) =
    linear_map_kernel_quotient_smul(src, dst, f, r,
        linear_map_kernel_quotient_project(src, dst, f, a))
} by {
    if is_linear_map(src, dst, f) {
        linear_map_kernel_quotient_smul_projection(src, dst, f, r, a)
    }
}

/// Addition of projected linear-map-kernel representatives is associative.
theorem linear_map_kernel_quotient_project_add_assoc[R: Ring, M: AddCommGroup, N: AddCommGroup](
    src: Module[R, M], dst: Module[R, N], f: M -> N, a: M, b: M, c: M
) {
    is_linear_map(src, dst, f) implies
    linear_map_kernel_quotient_add(src, dst, f,
        linear_map_kernel_quotient_add(src, dst, f,
            linear_map_kernel_quotient_project(src, dst, f, a),
            linear_map_kernel_quotient_project(src, dst, f, b)),
        linear_map_kernel_quotient_project(src, dst, f, c)) =
    linear_map_kernel_quotient_add(src, dst, f,
        linear_map_kernel_quotient_project(src, dst, f, a),
        linear_map_kernel_quotient_add(src, dst, f,
            linear_map_kernel_quotient_project(src, dst, f, b),
            linear_map_kernel_quotient_project(src, dst, f, c)))
} by {
    if is_linear_map(src, dst, f) {
        linear_map_kernel_quotient_add_assoc(src, dst, f, a, b, c)
    }
}

/// Addition of projected linear-map-kernel representatives is commutative.
theorem linear_map_kernel_quotient_project_add_comm[R: Ring, M: AddCommGroup, N: AddCommGroup](
    src: Module[R, M], dst: Module[R, N], f: M -> N, a: M, b: M
) {
    is_linear_map(src, dst, f) implies
    linear_map_kernel_quotient_add(src, dst, f,
        linear_map_kernel_quotient_project(src, dst, f, a),
        linear_map_kernel_quotient_project(src, dst, f, b)) =
    linear_map_kernel_quotient_add(src, dst, f,
        linear_map_kernel_quotient_project(src, dst, f, b),
        linear_map_kernel_quotient_project(src, dst, f, a))
} by {
    if is_linear_map(src, dst, f) {
        linear_map_kernel_quotient_add_comm(src, dst, f, a, b)
    }
}

/// The quotient zero is a left identity for projected linear-map-kernel representatives.
theorem linear_map_kernel_quotient_project_zero_add[R: Ring, M: AddCommGroup, N: AddCommGroup](
    src: Module[R, M], dst: Module[R, N], f: M -> N, a: M
) {
    is_linear_map(src, dst, f) implies
    linear_map_kernel_quotient_add(src, dst, f,
        linear_map_kernel_quotient_zero(src, dst, f),
        linear_map_kernel_quotient_project(src, dst, f, a)) =
        linear_map_kernel_quotient_project(src, dst, f, a)
} by {
    if is_linear_map(src, dst, f) {
        linear_map_kernel_quotient_add_zero_left_projection(src, dst, f, a)
    }
}

/// The quotient zero is a right identity for projected linear-map-kernel representatives.
theorem linear_map_kernel_quotient_project_add_zero[R: Ring, M: AddCommGroup, N: AddCommGroup](
    src: Module[R, M], dst: Module[R, N], f: M -> N, a: M
) {
    is_linear_map(src, dst, f) implies
    linear_map_kernel_quotient_add(src, dst, f,
        linear_map_kernel_quotient_project(src, dst, f, a),
        linear_map_kernel_quotient_zero(src, dst, f)) =
        linear_map_kernel_quotient_project(src, dst, f, a)
} by {
    if is_linear_map(src, dst, f) {
        linear_map_kernel_quotient_add_zero_right_projection(src, dst, f, a)
    }
}

/// The identity scalar fixes projected linear-map-kernel representatives.
theorem linear_map_kernel_quotient_project_smul_one[R: Ring, M: AddCommGroup, N: AddCommGroup](
    src: Module[R, M], dst: Module[R, N], f: M -> N, a: M
) {
    is_linear_map(src, dst, f) implies
    linear_map_kernel_quotient_smul(src, dst, f, R.1,
        linear_map_kernel_quotient_project(src, dst, f, a)) =
        linear_map_kernel_quotient_project(src, dst, f, a)
} by {
    if is_linear_map(src, dst, f) {
        linear_map_kernel_quotient_smul_one(src, dst, f, a)
    }
}

/// The canonical projection to a group-kernel quotient.
define group_hom_kernel_quotient_project[G: Group, H: Group](f: GroupHom[G, H], a: G) -> QuotientOver[G] {
    quotient_over_mk(kernel_quotient_relation(f.hom), a)
}

/// The canonical group-kernel projection is the quotient constructor.
theorem group_hom_kernel_quotient_project_eq_mk[G: Group, H: Group](f: GroupHom[G, H], a: G) {
    group_hom_kernel_quotient_project(f, a) = quotient_over_mk(kernel_quotient_relation(f.hom), a)
}

/// Equality of projected group-kernel representatives is the kernel relation.
theorem group_hom_kernel_quotient_project_eq_iff_kernel_relation[G: Group, H: Group](
    f: GroupHom[G, H], a: G, b: G
) {
    (group_hom_kernel_quotient_project(f, a) = group_hom_kernel_quotient_project(f, b)) =
        kernel_relation(f.hom, a, b)
} by {
    group_hom_kernel_quotient_project_eq_mk(f, a)
    group_hom_kernel_quotient_project_eq_mk(f, b)
    quotient_over_mk_eq_iff_rel(kernel_quotient_relation(f.hom), a, b)
    kernel_quotient_relation_rel(f.hom)
}

/// The homomorphism out of the group-kernel quotient induced by `f`.
define group_hom_kernel_quotient_induced[G: Group, H: Group](f: GroupHom[G, H]) ->
    QuotientOver[G] -> H {
    quotient_over_lift_to(f.hom)
}

/// The induced map agrees with `f` on canonical group-kernel projections.
theorem group_hom_kernel_quotient_induced_project[G: Group, H: Group](f: GroupHom[G, H], a: G) {
    group_hom_kernel_quotient_induced(f, group_hom_kernel_quotient_project(f, a)) = f.hom(a)
} by {
    kernel_relation_respects(f.hom)
    group_hom_kernel_quotient_project_eq_mk(f, a)
    quotient_over_lift_to_projection(kernel_quotient_relation(f.hom), f.hom, a)
}

/// The induced map out of the group-kernel quotient separates projections of unrelated representatives.
theorem group_hom_kernel_quotient_induced_injective[G: Group, H: Group](
    f: GroupHom[G, H], a: G, b: G
) {
    group_hom_kernel_quotient_induced(f, group_hom_kernel_quotient_project(f, a)) =
        group_hom_kernel_quotient_induced(f, group_hom_kernel_quotient_project(f, b))
    implies group_hom_kernel_quotient_project(f, a) = group_hom_kernel_quotient_project(f, b)
} by {
    if group_hom_kernel_quotient_induced(f, group_hom_kernel_quotient_project(f, a)) =
        group_hom_kernel_quotient_induced(f, group_hom_kernel_quotient_project(f, b)) {
        group_hom_kernel_quotient_induced_project(f, a)
        group_hom_kernel_quotient_induced_project(f, b)
        f.hom(a) = f.hom(b)
        kernel_relation(f.hom, a, b) = (f.hom(a) = f.hom(b))
        kernel_relation(f.hom, a, b)
        group_hom_kernel_quotient_project_eq_iff_kernel_relation(f, a, b)
        (group_hom_kernel_quotient_project(f, a) = group_hom_kernel_quotient_project(f, b)) =
            kernel_relation(f.hom, a, b)
        group_hom_kernel_quotient_project(f, a) = group_hom_kernel_quotient_project(f, b)
    }
}

/// The canonical group-kernel projection preserves the identity.
theorem group_hom_kernel_quotient_project_one[G: Group, H: Group](f: GroupHom[G, H]) {
    group_hom_kernel_quotient_project(f, G.1) = group_hom_kernel_quotient_one(f)
} by {
    group_hom_kernel_quotient_one_projection(f)
}

/// The canonical group-kernel projection preserves multiplication.
theorem group_hom_kernel_quotient_project_mul[G: Group, H: Group](f: GroupHom[G, H], a: G, b: G) {
    group_hom_kernel_quotient_project(f, a * b) =
    group_hom_kernel_quotient_mul(f,
        group_hom_kernel_quotient_project(f, a),
        group_hom_kernel_quotient_project(f, b))
} by {
    group_hom_kernel_quotient_mul_projection(f, a, b)
    group_hom_kernel_quotient_project(f, a * b) =
        quotient_over_mk(kernel_quotient_relation(f.hom), a * b)
    group_hom_kernel_quotient_project(f, a) =
        quotient_over_mk(kernel_quotient_relation(f.hom), a)
    group_hom_kernel_quotient_project(f, b) =
        quotient_over_mk(kernel_quotient_relation(f.hom), b)
}

/// The canonical group-kernel projection preserves inversion.
theorem group_hom_kernel_quotient_project_inverse[G: Group, H: Group](f: GroupHom[G, H], a: G) {
    group_hom_kernel_quotient_project(f, a.inverse) =
    group_hom_kernel_quotient_inverse(f, group_hom_kernel_quotient_project(f, a))
} by {
    group_hom_kernel_quotient_inverse_projection(f, a)
    group_hom_kernel_quotient_project(f, a.inverse) =
        quotient_over_mk(kernel_quotient_relation(f.hom), a.inverse)
    group_hom_kernel_quotient_project(f, a) =
        quotient_over_mk(kernel_quotient_relation(f.hom), a)
}

/// The induced map out of the group-kernel quotient preserves multiplication.
theorem group_hom_kernel_quotient_induced_mul[G: Group, H: Group](f: GroupHom[G, H], a: G, b: G) {
    group_hom_kernel_quotient_induced(f,
        group_hom_kernel_quotient_mul(f,
            group_hom_kernel_quotient_project(f, a),
            group_hom_kernel_quotient_project(f, b))) =
        group_hom_kernel_quotient_induced(f, group_hom_kernel_quotient_project(f, a)) *
        group_hom_kernel_quotient_induced(f, group_hom_kernel_quotient_project(f, b))
} by {
    group_hom_kernel_quotient_project_mul(f, a, b)
    group_hom_kernel_quotient_induced_project(f, a * b)
    group_hom_kernel_quotient_induced_project(f, a)
    group_hom_kernel_quotient_induced_project(f, b)
    group_hom_mul(f, a, b)
}

/// The induced map out of the group-kernel quotient preserves the identity.
theorem group_hom_kernel_quotient_induced_one[G: Group, H: Group](f: GroupHom[G, H]) {
    group_hom_kernel_quotient_induced(f, group_hom_kernel_quotient_one(f)) = H.1
} by {
    group_hom_kernel_quotient_project_one(f)
    group_hom_kernel_quotient_induced_project(f, G.1)
    group_hom_one(f)
}

/// The induced map out of the group-kernel quotient preserves inversion.
theorem group_hom_kernel_quotient_induced_inverse[G: Group, H: Group](f: GroupHom[G, H], a: G) {
    group_hom_kernel_quotient_induced(f,
        group_hom_kernel_quotient_inverse(f, group_hom_kernel_quotient_project(f, a))) =
        group_hom_kernel_quotient_induced(f, group_hom_kernel_quotient_project(f, a)).inverse
} by {
    group_hom_kernel_quotient_project_inverse(f, a)
    group_hom_kernel_quotient_induced_project(f, a.inverse)
    group_hom_kernel_quotient_induced_project(f, a)
    group_hom_inv(f, a)
}

/// Every value of `f` is attained by the induced map out of the group-kernel quotient.
theorem group_hom_kernel_quotient_induced_surjective_onto_image[G: Group, H: Group](
    f: GroupHom[G, H], a: G
) {
    exists(q: QuotientOver[G]) {
        group_hom_kernel_quotient_induced(f, q) = f.hom(a)
    }
} by {
    group_hom_kernel_quotient_induced_project(f, a)
    group_hom_kernel_quotient_induced(f, group_hom_kernel_quotient_project(f, a)) = f.hom(a)
}

/// The zeroth power of a projected group-kernel representative is the projected identity.
theorem group_hom_kernel_quotient_project_pow_zero[G: Group, H: Group](f: GroupHom[G, H], a: G) {
    group_hom_kernel_quotient_pow(f, group_hom_kernel_quotient_project(f, a), Nat.0) =
        group_hom_kernel_quotient_project(f, a.pow(Nat.0))
} by {
    group_hom_kernel_quotient_pow_mk_zero(f, a)
}

/// Powers of projected group-kernel representatives agree with projected powers.
theorem group_hom_kernel_quotient_project_pow[G: Group, H: Group](f: GroupHom[G, H], a: G, n: Nat) {
    group_hom_kernel_quotient_pow(f, group_hom_kernel_quotient_project(f, a), n) =
        group_hom_kernel_quotient_project(f, a.pow(n))
} by {
    group_hom_kernel_quotient_pow_mk(f, a, n)
}

/// The first power of a projected group-kernel representative is the projected representative.
theorem group_hom_kernel_quotient_project_pow_one[G: Group, H: Group](f: GroupHom[G, H], a: G) {
    group_hom_kernel_quotient_pow(f, group_hom_kernel_quotient_project(f, a), Nat.1) =
        group_hom_kernel_quotient_project(f, a)
} by {
    group_hom_kernel_quotient_project_pow(f, a, Nat.1)
    a.pow(Nat.1) = a
}

/// Quotient one raised to a natural power is quotient one in a projected group-kernel quotient.
theorem group_hom_kernel_quotient_project_one_pow[G: Group, H: Group](f: GroupHom[G, H], n: Nat) {
    group_hom_kernel_quotient_pow(f, group_hom_kernel_quotient_one(f), n) =
        group_hom_kernel_quotient_one(f)
} by {
    group_hom_kernel_quotient_one_pow(f, n)
}

/// Multiplying projected group-kernel quotient powers adds their exponents.
theorem group_hom_kernel_quotient_project_pow_add[G: Group, H: Group](
    f: GroupHom[G, H], a: G, n: Nat, m: Nat
) {
    group_hom_kernel_quotient_mul(f,
        group_hom_kernel_quotient_pow(f, group_hom_kernel_quotient_project(f, a), n),
        group_hom_kernel_quotient_pow(f, group_hom_kernel_quotient_project(f, a), m)) =
    group_hom_kernel_quotient_pow(f, group_hom_kernel_quotient_project(f, a), n + m)
} by {
    group_hom_kernel_quotient_pow_add(f, a, n, m)
}

/// Raising a projected group-kernel quotient power to a power multiplies the exponents.
theorem group_hom_kernel_quotient_project_pow_pow[G: Group, H: Group](
    f: GroupHom[G, H], a: G, n: Nat, m: Nat
) {
    group_hom_kernel_quotient_pow(f,
        group_hom_kernel_quotient_pow(f, group_hom_kernel_quotient_project(f, a), n),
        m) =
    group_hom_kernel_quotient_pow(f, group_hom_kernel_quotient_project(f, a), n * m)
} by {
    group_hom_kernel_quotient_pow_pow(f, a, n, m)
}

/// Multiplication of projected group-kernel representatives is associative.
theorem group_hom_kernel_quotient_project_mul_assoc[G: Group, H: Group](f: GroupHom[G, H], a: G, b: G, c: G) {
    group_hom_kernel_quotient_mul(f,
        group_hom_kernel_quotient_mul(f,
            group_hom_kernel_quotient_project(f, a),
            group_hom_kernel_quotient_project(f, b)),
        group_hom_kernel_quotient_project(f, c)) =
    group_hom_kernel_quotient_mul(f,
        group_hom_kernel_quotient_project(f, a),
        group_hom_kernel_quotient_mul(f,
            group_hom_kernel_quotient_project(f, b),
            group_hom_kernel_quotient_project(f, c)))
} by {
    group_hom_kernel_quotient_mul_assoc(f, a, b, c)
}

/// The quotient identity is a left identity for projected group-kernel representatives.
theorem group_hom_kernel_quotient_project_one_mul[G: Group, H: Group](f: GroupHom[G, H], a: G) {
    group_hom_kernel_quotient_mul(f,
        group_hom_kernel_quotient_one(f),
        group_hom_kernel_quotient_project(f, a)) =
        group_hom_kernel_quotient_project(f, a)
} by {
    group_hom_kernel_quotient_mul_one_left_projection(f, a)
}

/// The quotient identity is a right identity for projected group-kernel representatives.
theorem group_hom_kernel_quotient_project_mul_one[G: Group, H: Group](f: GroupHom[G, H], a: G) {
    group_hom_kernel_quotient_mul(f,
        group_hom_kernel_quotient_project(f, a),
        group_hom_kernel_quotient_one(f)) =
        group_hom_kernel_quotient_project(f, a)
} by {
    group_hom_kernel_quotient_mul_one_right_projection(f, a)
}

/// Inversion twice fixes a projected group-kernel representative.
theorem group_hom_kernel_quotient_project_inverse_inverse[G: Group, H: Group](f: GroupHom[G, H], a: G) {
    group_hom_kernel_quotient_inverse(f,
        group_hom_kernel_quotient_inverse(f, group_hom_kernel_quotient_project(f, a))) =
        group_hom_kernel_quotient_project(f, a)
} by {
    group_hom_kernel_quotient_inverse_inverse(f, a)
}

/// The quotient inverse is a left inverse for projected group-kernel representatives.
theorem group_hom_kernel_quotient_project_inverse_left[G: Group, H: Group](f: GroupHom[G, H], a: G) {
    group_hom_kernel_quotient_mul(f,
        group_hom_kernel_quotient_inverse(f, group_hom_kernel_quotient_project(f, a)),
        group_hom_kernel_quotient_project(f, a)) =
        group_hom_kernel_quotient_one(f)
} by {
    group_hom_kernel_quotient_inverse_left(f, a)
    group_hom_kernel_quotient_one_projection(f)
}

/// The quotient inverse is a right inverse for projected group-kernel representatives.
theorem group_hom_kernel_quotient_project_inverse_right[G: Group, H: Group](f: GroupHom[G, H], a: G) {
    group_hom_kernel_quotient_mul(f,
        group_hom_kernel_quotient_project(f, a),
        group_hom_kernel_quotient_inverse(f, group_hom_kernel_quotient_project(f, a))) =
        group_hom_kernel_quotient_one(f)
} by {
    group_hom_kernel_quotient_inverse_right(f, a)
    group_hom_kernel_quotient_one_projection(f)
}

/// Multiplication by a projected quotient inverse cancels on the left.
theorem group_hom_kernel_quotient_project_inverse_mul_cancel_left[G: Group, H: Group](
    f: GroupHom[G, H], a: G, b: G
) {
    group_hom_kernel_quotient_mul(f,
        group_hom_kernel_quotient_inverse(f, group_hom_kernel_quotient_project(f, a)),
        group_hom_kernel_quotient_mul(f,
            group_hom_kernel_quotient_project(f, a),
            group_hom_kernel_quotient_project(f, b))) =
        group_hom_kernel_quotient_project(f, b)
} by {
    group_hom_kernel_quotient_inverse_mul_cancel_left(f, a, b)
}

/// Multiplication by a projected quotient inverse cancels on the right.
theorem group_hom_kernel_quotient_project_mul_inverse_cancel_right[G: Group, H: Group](
    f: GroupHom[G, H], a: G, b: G
) {
    group_hom_kernel_quotient_mul(f,
        group_hom_kernel_quotient_mul(f,
            group_hom_kernel_quotient_project(f, b),
            group_hom_kernel_quotient_project(f, a)),
        group_hom_kernel_quotient_inverse(f, group_hom_kernel_quotient_project(f, a))) =
        group_hom_kernel_quotient_project(f, b)
} by {
    group_hom_kernel_quotient_mul_inverse_cancel_right(f, a, b)
}

/// Equal group-kernel quotient products with a common projected left factor have equal right factors.
theorem group_hom_kernel_quotient_project_mul_left_cancel[G: Group, H: Group](
    f: GroupHom[G, H], a: G, b: G, c: G
) {
    group_hom_kernel_quotient_mul(f,
        group_hom_kernel_quotient_project(f, a),
        group_hom_kernel_quotient_project(f, b)) =
    group_hom_kernel_quotient_mul(f,
        group_hom_kernel_quotient_project(f, a),
        group_hom_kernel_quotient_project(f, c)) implies
    group_hom_kernel_quotient_project(f, b) = group_hom_kernel_quotient_project(f, c)
} by {
    if group_hom_kernel_quotient_mul(f,
        group_hom_kernel_quotient_project(f, a),
        group_hom_kernel_quotient_project(f, b)) =
    group_hom_kernel_quotient_mul(f,
        group_hom_kernel_quotient_project(f, a),
        group_hom_kernel_quotient_project(f, c)) {
        group_hom_kernel_quotient_project_inverse_mul_cancel_left(f, a, b)
        group_hom_kernel_quotient_mul(f,
            group_hom_kernel_quotient_inverse(f, group_hom_kernel_quotient_project(f, a)),
            group_hom_kernel_quotient_mul(f,
                group_hom_kernel_quotient_project(f, a),
                group_hom_kernel_quotient_project(f, b))) =
            group_hom_kernel_quotient_project(f, b)
        group_hom_kernel_quotient_project_inverse_mul_cancel_left(f, a, c)
        group_hom_kernel_quotient_mul(f,
            group_hom_kernel_quotient_inverse(f, group_hom_kernel_quotient_project(f, a)),
            group_hom_kernel_quotient_mul(f,
                group_hom_kernel_quotient_project(f, a),
                group_hom_kernel_quotient_project(f, c))) =
            group_hom_kernel_quotient_project(f, c)
        group_hom_kernel_quotient_project(f, b) = group_hom_kernel_quotient_project(f, c)
    }
}

/// Equal group-kernel quotient products with a common projected right factor have equal left factors.
theorem group_hom_kernel_quotient_project_mul_right_cancel[G: Group, H: Group](
    f: GroupHom[G, H], a: G, b: G, c: G
) {
    group_hom_kernel_quotient_mul(f,
        group_hom_kernel_quotient_project(f, b),
        group_hom_kernel_quotient_project(f, a)) =
    group_hom_kernel_quotient_mul(f,
        group_hom_kernel_quotient_project(f, c),
        group_hom_kernel_quotient_project(f, a)) implies
    group_hom_kernel_quotient_project(f, b) = group_hom_kernel_quotient_project(f, c)
} by {
    if group_hom_kernel_quotient_mul(f,
        group_hom_kernel_quotient_project(f, b),
        group_hom_kernel_quotient_project(f, a)) =
    group_hom_kernel_quotient_mul(f,
        group_hom_kernel_quotient_project(f, c),
        group_hom_kernel_quotient_project(f, a)) {
        group_hom_kernel_quotient_project_mul_inverse_cancel_right(f, a, b)
        group_hom_kernel_quotient_mul(f,
            group_hom_kernel_quotient_mul(f,
                group_hom_kernel_quotient_project(f, b),
                group_hom_kernel_quotient_project(f, a)),
            group_hom_kernel_quotient_inverse(f, group_hom_kernel_quotient_project(f, a))) =
            group_hom_kernel_quotient_project(f, b)
        group_hom_kernel_quotient_project_mul_inverse_cancel_right(f, a, c)
        group_hom_kernel_quotient_mul(f,
            group_hom_kernel_quotient_mul(f,
                group_hom_kernel_quotient_project(f, c),
                group_hom_kernel_quotient_project(f, a)),
            group_hom_kernel_quotient_inverse(f, group_hom_kernel_quotient_project(f, a))) =
            group_hom_kernel_quotient_project(f, c)
        group_hom_kernel_quotient_project(f, b) = group_hom_kernel_quotient_project(f, c)
    }
}

/// Equality after multiplying projected group-kernel representatives on the left is exactly equality before multiplying.
theorem group_hom_kernel_quotient_project_mul_left_cancel_iff[G: Group, H: Group](
    f: GroupHom[G, H], a: G, b: G, c: G
) {
    (group_hom_kernel_quotient_mul(f,
        group_hom_kernel_quotient_project(f, a),
        group_hom_kernel_quotient_project(f, b)) =
    group_hom_kernel_quotient_mul(f,
        group_hom_kernel_quotient_project(f, a),
        group_hom_kernel_quotient_project(f, c))) =
    (group_hom_kernel_quotient_project(f, b) = group_hom_kernel_quotient_project(f, c))
} by {
    if group_hom_kernel_quotient_mul(f,
        group_hom_kernel_quotient_project(f, a),
        group_hom_kernel_quotient_project(f, b)) =
    group_hom_kernel_quotient_mul(f,
        group_hom_kernel_quotient_project(f, a),
        group_hom_kernel_quotient_project(f, c)) {
        group_hom_kernel_quotient_project_mul_left_cancel(f, a, b, c)
    }
    if group_hom_kernel_quotient_project(f, b) = group_hom_kernel_quotient_project(f, c) {
        group_hom_kernel_quotient_mul(f,
            group_hom_kernel_quotient_project(f, a),
            group_hom_kernel_quotient_project(f, b)) =
        group_hom_kernel_quotient_mul(f,
            group_hom_kernel_quotient_project(f, a),
            group_hom_kernel_quotient_project(f, c))
    }
}

/// Equality after multiplying projected group-kernel representatives on the right is exactly equality before multiplying.
theorem group_hom_kernel_quotient_project_mul_right_cancel_iff[G: Group, H: Group](
    f: GroupHom[G, H], a: G, b: G, c: G
) {
    (group_hom_kernel_quotient_mul(f,
        group_hom_kernel_quotient_project(f, b),
        group_hom_kernel_quotient_project(f, a)) =
    group_hom_kernel_quotient_mul(f,
        group_hom_kernel_quotient_project(f, c),
        group_hom_kernel_quotient_project(f, a))) =
    (group_hom_kernel_quotient_project(f, b) = group_hom_kernel_quotient_project(f, c))
} by {
    if group_hom_kernel_quotient_mul(f,
        group_hom_kernel_quotient_project(f, b),
        group_hom_kernel_quotient_project(f, a)) =
    group_hom_kernel_quotient_mul(f,
        group_hom_kernel_quotient_project(f, c),
        group_hom_kernel_quotient_project(f, a)) {
        group_hom_kernel_quotient_project_mul_right_cancel(f, a, b, c)
    }
    if group_hom_kernel_quotient_project(f, b) = group_hom_kernel_quotient_project(f, c) {
        group_hom_kernel_quotient_mul(f,
            group_hom_kernel_quotient_project(f, b),
            group_hom_kernel_quotient_project(f, a)) =
        group_hom_kernel_quotient_mul(f,
            group_hom_kernel_quotient_project(f, c),
            group_hom_kernel_quotient_project(f, a))
    }
}

/// Equality after multiplying projected group-kernel representatives on the left is exactly the kernel relation.
theorem group_hom_kernel_quotient_project_mul_left_eq_iff_kernel_relation[G: Group, H: Group](
    f: GroupHom[G, H], a: G, b: G, c: G
) {
    (group_hom_kernel_quotient_mul(f,
        group_hom_kernel_quotient_project(f, a),
        group_hom_kernel_quotient_project(f, b)) =
    group_hom_kernel_quotient_mul(f,
        group_hom_kernel_quotient_project(f, a),
        group_hom_kernel_quotient_project(f, c))) =
    kernel_relation(f.hom, b, c)
} by {
    group_hom_kernel_quotient_project_mul_left_cancel_iff(f, a, b, c)
    (group_hom_kernel_quotient_mul(f,
        group_hom_kernel_quotient_project(f, a),
        group_hom_kernel_quotient_project(f, b)) =
    group_hom_kernel_quotient_mul(f,
        group_hom_kernel_quotient_project(f, a),
        group_hom_kernel_quotient_project(f, c))) =
    (group_hom_kernel_quotient_project(f, b) = group_hom_kernel_quotient_project(f, c))
    group_hom_kernel_quotient_project_eq_iff_kernel_relation(f, b, c)
    (group_hom_kernel_quotient_project(f, b) = group_hom_kernel_quotient_project(f, c)) =
        kernel_relation(f.hom, b, c)
    (group_hom_kernel_quotient_mul(f,
        group_hom_kernel_quotient_project(f, a),
        group_hom_kernel_quotient_project(f, b)) =
    group_hom_kernel_quotient_mul(f,
        group_hom_kernel_quotient_project(f, a),
        group_hom_kernel_quotient_project(f, c))) =
    kernel_relation(f.hom, b, c)
}

/// Equality after multiplying projected group-kernel representatives on the right is exactly the kernel relation.
theorem group_hom_kernel_quotient_project_mul_right_eq_iff_kernel_relation[G: Group, H: Group](
    f: GroupHom[G, H], a: G, b: G, c: G
) {
    (group_hom_kernel_quotient_mul(f,
        group_hom_kernel_quotient_project(f, b),
        group_hom_kernel_quotient_project(f, a)) =
    group_hom_kernel_quotient_mul(f,
        group_hom_kernel_quotient_project(f, c),
        group_hom_kernel_quotient_project(f, a))) =
    kernel_relation(f.hom, b, c)
} by {
    group_hom_kernel_quotient_project_mul_right_cancel_iff(f, a, b, c)
    (group_hom_kernel_quotient_mul(f,
        group_hom_kernel_quotient_project(f, b),
        group_hom_kernel_quotient_project(f, a)) =
    group_hom_kernel_quotient_mul(f,
        group_hom_kernel_quotient_project(f, c),
        group_hom_kernel_quotient_project(f, a))) =
    (group_hom_kernel_quotient_project(f, b) = group_hom_kernel_quotient_project(f, c))
    group_hom_kernel_quotient_project_eq_iff_kernel_relation(f, b, c)
    (group_hom_kernel_quotient_project(f, b) = group_hom_kernel_quotient_project(f, c)) =
        kernel_relation(f.hom, b, c)
    (group_hom_kernel_quotient_mul(f,
        group_hom_kernel_quotient_project(f, b),
        group_hom_kernel_quotient_project(f, a)) =
    group_hom_kernel_quotient_mul(f,
        group_hom_kernel_quotient_project(f, c),
        group_hom_kernel_quotient_project(f, a))) =
    kernel_relation(f.hom, b, c)
}

/// Multiplication of projected group-kernel representatives is commutative in the commutative case.
theorem group_hom_kernel_quotient_project_mul_comm[G: CommGroup, H: CommGroup](
    f: GroupHom[G, H], a: G, b: G
) {
    group_hom_kernel_quotient_mul(f,
        group_hom_kernel_quotient_project(f, a),
        group_hom_kernel_quotient_project(f, b)) =
    group_hom_kernel_quotient_mul(f,
        group_hom_kernel_quotient_project(f, b),
        group_hom_kernel_quotient_project(f, a))
} by {
    group_hom_kernel_quotient_mul_comm(f, a, b)
}

/// The canonical projection to a ring-kernel quotient.
define ring_hom_kernel_quotient_project[R: Ring, S: Ring](f: RingHom[R, S], a: R) -> QuotientOver[R] {
    quotient_over_mk(kernel_quotient_relation(f.hom), a)
}

/// The canonical ring-kernel projection is the quotient constructor.
theorem ring_hom_kernel_quotient_project_eq_mk[R: Ring, S: Ring](f: RingHom[R, S], a: R) {
    ring_hom_kernel_quotient_project(f, a) = quotient_over_mk(kernel_quotient_relation(f.hom), a)
}

/// Equality of projected ring-kernel representatives is the kernel relation.
theorem ring_hom_kernel_quotient_project_eq_iff_kernel_relation[R: Ring, S: Ring](
    f: RingHom[R, S], a: R, b: R
) {
    (ring_hom_kernel_quotient_project(f, a) = ring_hom_kernel_quotient_project(f, b)) =
        kernel_relation(f.hom, a, b)
} by {
    ring_hom_kernel_quotient_project_eq_mk(f, a)
    ring_hom_kernel_quotient_project_eq_mk(f, b)
    quotient_over_mk_eq_iff_rel(kernel_quotient_relation(f.hom), a, b)
    kernel_quotient_relation_rel(f.hom)
}

/// The homomorphism out of the ring-kernel quotient induced by `f`.
define ring_hom_kernel_quotient_induced[R: Ring, S: Ring](f: RingHom[R, S]) ->
    QuotientOver[R] -> S {
    quotient_over_lift_to(f.hom)
}

/// The induced map agrees with `f` on canonical ring-kernel projections.
theorem ring_hom_kernel_quotient_induced_project[R: Ring, S: Ring](f: RingHom[R, S], a: R) {
    ring_hom_kernel_quotient_induced(f, ring_hom_kernel_quotient_project(f, a)) = f.hom(a)
} by {
    kernel_relation_respects(f.hom)
    ring_hom_kernel_quotient_project_eq_mk(f, a)
    quotient_over_lift_to_projection(kernel_quotient_relation(f.hom), f.hom, a)
}

/// The induced map out of the ring-kernel quotient separates projections of unrelated representatives.
theorem ring_hom_kernel_quotient_induced_injective[R: Ring, S: Ring](
    f: RingHom[R, S], a: R, b: R
) {
    ring_hom_kernel_quotient_induced(f, ring_hom_kernel_quotient_project(f, a)) =
        ring_hom_kernel_quotient_induced(f, ring_hom_kernel_quotient_project(f, b))
    implies ring_hom_kernel_quotient_project(f, a) = ring_hom_kernel_quotient_project(f, b)
} by {
    if ring_hom_kernel_quotient_induced(f, ring_hom_kernel_quotient_project(f, a)) =
        ring_hom_kernel_quotient_induced(f, ring_hom_kernel_quotient_project(f, b)) {
        ring_hom_kernel_quotient_induced_project(f, a)
        ring_hom_kernel_quotient_induced_project(f, b)
        f.hom(a) = f.hom(b)
        kernel_relation(f.hom, a, b) = (f.hom(a) = f.hom(b))
        kernel_relation(f.hom, a, b)
        ring_hom_kernel_quotient_project_eq_iff_kernel_relation(f, a, b)
        (ring_hom_kernel_quotient_project(f, a) = ring_hom_kernel_quotient_project(f, b)) =
            kernel_relation(f.hom, a, b)
        ring_hom_kernel_quotient_project(f, a) = ring_hom_kernel_quotient_project(f, b)
    }
}

/// The canonical ring-kernel projection preserves zero.
theorem ring_hom_kernel_quotient_project_zero[R: Ring, S: Ring](f: RingHom[R, S]) {
    ring_hom_kernel_quotient_project(f, R.0) = ring_hom_kernel_quotient_zero(f)
} by {
    ring_hom_kernel_quotient_zero_projection(f)
}

/// The canonical ring-kernel projection preserves one.
theorem ring_hom_kernel_quotient_project_one[R: Ring, S: Ring](f: RingHom[R, S]) {
    ring_hom_kernel_quotient_project(f, R.1) = ring_hom_kernel_quotient_one(f)
} by {
    ring_hom_kernel_quotient_one_projection(f)
}

/// The canonical ring-kernel projection preserves addition.
theorem ring_hom_kernel_quotient_project_add[R: Ring, S: Ring](f: RingHom[R, S], a: R, b: R) {
    ring_hom_kernel_quotient_project(f, a + b) =
    ring_hom_kernel_quotient_add(f,
        ring_hom_kernel_quotient_project(f, a),
        ring_hom_kernel_quotient_project(f, b))
} by {
    ring_hom_kernel_quotient_add_projection(f, a, b)
    ring_hom_kernel_quotient_project(f, a + b) =
        quotient_over_mk(kernel_quotient_relation(f.hom), a + b)
    ring_hom_kernel_quotient_project(f, a) =
        quotient_over_mk(kernel_quotient_relation(f.hom), a)
    ring_hom_kernel_quotient_project(f, b) =
        quotient_over_mk(kernel_quotient_relation(f.hom), b)
}

/// The canonical ring-kernel projection preserves multiplication.
theorem ring_hom_kernel_quotient_project_mul[R: Ring, S: Ring](f: RingHom[R, S], a: R, b: R) {
    ring_hom_kernel_quotient_project(f, a * b) =
    ring_hom_kernel_quotient_mul(f,
        ring_hom_kernel_quotient_project(f, a),
        ring_hom_kernel_quotient_project(f, b))
} by {
    ring_hom_kernel_quotient_mul_projection(f, a, b)
    ring_hom_kernel_quotient_project(f, a * b) =
        quotient_over_mk(kernel_quotient_relation(f.hom), a * b)
    ring_hom_kernel_quotient_project(f, a) =
        quotient_over_mk(kernel_quotient_relation(f.hom), a)
    ring_hom_kernel_quotient_project(f, b) =
        quotient_over_mk(kernel_quotient_relation(f.hom), b)
}

/// The canonical ring-kernel projection preserves negation.
theorem ring_hom_kernel_quotient_project_neg[R: Ring, S: Ring](f: RingHom[R, S], a: R) {
    ring_hom_kernel_quotient_project(f, -a) =
    ring_hom_kernel_quotient_neg(f, ring_hom_kernel_quotient_project(f, a))
} by {
    ring_hom_kernel_quotient_neg_projection(f, a)
    ring_hom_kernel_quotient_project(f, -a) =
        quotient_over_mk(kernel_quotient_relation(f.hom), -a)
    ring_hom_kernel_quotient_project(f, a) =
        quotient_over_mk(kernel_quotient_relation(f.hom), a)
}

/// The induced map out of the ring-kernel quotient preserves addition.
theorem ring_hom_kernel_quotient_induced_add[R: Ring, S: Ring](f: RingHom[R, S], a: R, b: R) {
    ring_hom_kernel_quotient_induced(f,
        ring_hom_kernel_quotient_add(f,
            ring_hom_kernel_quotient_project(f, a),
            ring_hom_kernel_quotient_project(f, b))) =
        ring_hom_kernel_quotient_induced(f, ring_hom_kernel_quotient_project(f, a)) +
        ring_hom_kernel_quotient_induced(f, ring_hom_kernel_quotient_project(f, b))
} by {
    ring_hom_kernel_quotient_project_add(f, a, b)
    ring_hom_kernel_quotient_induced_project(f, a + b)
    ring_hom_kernel_quotient_induced_project(f, a)
    ring_hom_kernel_quotient_induced_project(f, b)
    ring_hom_add(f, a, b)
}

/// The induced map out of the ring-kernel quotient preserves multiplication.
theorem ring_hom_kernel_quotient_induced_mul[R: Ring, S: Ring](f: RingHom[R, S], a: R, b: R) {
    ring_hom_kernel_quotient_induced(f,
        ring_hom_kernel_quotient_mul(f,
            ring_hom_kernel_quotient_project(f, a),
            ring_hom_kernel_quotient_project(f, b))) =
        ring_hom_kernel_quotient_induced(f, ring_hom_kernel_quotient_project(f, a)) *
        ring_hom_kernel_quotient_induced(f, ring_hom_kernel_quotient_project(f, b))
} by {
    ring_hom_kernel_quotient_project_mul(f, a, b)
    ring_hom_kernel_quotient_induced_project(f, a * b)
    ring_hom_kernel_quotient_induced_project(f, a)
    ring_hom_kernel_quotient_induced_project(f, b)
    ring_hom_mul(f, a, b)
}

/// The induced map out of the ring-kernel quotient preserves negation.
theorem ring_hom_kernel_quotient_induced_neg[R: Ring, S: Ring](f: RingHom[R, S], a: R) {
    ring_hom_kernel_quotient_induced(f,
        ring_hom_kernel_quotient_neg(f, ring_hom_kernel_quotient_project(f, a))) =
        -ring_hom_kernel_quotient_induced(f, ring_hom_kernel_quotient_project(f, a))
} by {
    ring_hom_kernel_quotient_project_neg(f, a)
    ring_hom_kernel_quotient_induced_project(f, -a)
    ring_hom_kernel_quotient_induced_project(f, a)
    ring_hom_neg(f, a)
}

/// The induced map out of the ring-kernel quotient preserves zero.
theorem ring_hom_kernel_quotient_induced_zero[R: Ring, S: Ring](f: RingHom[R, S]) {
    ring_hom_kernel_quotient_induced(f, ring_hom_kernel_quotient_zero(f)) = S.0
} by {
    ring_hom_kernel_quotient_project_zero(f)
    ring_hom_kernel_quotient_induced_project(f, R.0)
    ring_hom_zero(f)
}

/// The induced map out of the ring-kernel quotient preserves one.
theorem ring_hom_kernel_quotient_induced_one[R: Ring, S: Ring](f: RingHom[R, S]) {
    ring_hom_kernel_quotient_induced(f, ring_hom_kernel_quotient_one(f)) = S.1
} by {
    ring_hom_kernel_quotient_project_one(f)
    ring_hom_kernel_quotient_induced_project(f, R.1)
    ring_hom_one(f)
}

/// Every value of `f` is attained by the induced map out of the ring-kernel quotient.
theorem ring_hom_kernel_quotient_induced_surjective_onto_image[R: Ring, S: Ring](
    f: RingHom[R, S], a: R
) {
    exists(q: QuotientOver[R]) {
        ring_hom_kernel_quotient_induced(f, q) = f.hom(a)
    }
} by {
    ring_hom_kernel_quotient_induced_project(f, a)
    ring_hom_kernel_quotient_induced(f, ring_hom_kernel_quotient_project(f, a)) = f.hom(a)
}

/// The canonical ring-kernel projection preserves subtraction.
theorem ring_hom_kernel_quotient_project_sub[R: Ring, S: Ring](f: RingHom[R, S], a: R, b: R) {
    ring_hom_kernel_quotient_project(f, a - b) =
    ring_hom_kernel_quotient_sub(f,
        ring_hom_kernel_quotient_project(f, a),
        ring_hom_kernel_quotient_project(f, b))
} by {
    ring_hom_kernel_quotient_sub_projection(f, a, b)
    ring_hom_kernel_quotient_project(f, a - b) =
        quotient_over_mk(kernel_quotient_relation(f.hom), a - b)
    ring_hom_kernel_quotient_project(f, a) =
        quotient_over_mk(kernel_quotient_relation(f.hom), a)
    ring_hom_kernel_quotient_project(f, b) =
        quotient_over_mk(kernel_quotient_relation(f.hom), b)
}

/// The zeroth power of a projected ring-kernel representative is the projected identity.
theorem ring_hom_kernel_quotient_project_pow_zero[R: Ring, S: Ring](f: RingHom[R, S], a: R) {
    ring_hom_kernel_quotient_pow(f, ring_hom_kernel_quotient_project(f, a), Nat.0) =
        ring_hom_kernel_quotient_project(f, a.pow(Nat.0))
} by {
    ring_hom_kernel_quotient_pow_mk_zero(f, a)
}

/// Powers of projected ring-kernel representatives agree with projected powers.
theorem ring_hom_kernel_quotient_project_pow[R: Ring, S: Ring](f: RingHom[R, S], a: R, n: Nat) {
    ring_hom_kernel_quotient_pow(f, ring_hom_kernel_quotient_project(f, a), n) =
        ring_hom_kernel_quotient_project(f, a.pow(n))
} by {
    ring_hom_kernel_quotient_pow_mk(f, a, n)
}

/// The first power of a projected ring-kernel representative is the projected representative.
theorem ring_hom_kernel_quotient_project_pow_one[R: Ring, S: Ring](f: RingHom[R, S], a: R) {
    ring_hom_kernel_quotient_pow(f, ring_hom_kernel_quotient_project(f, a), Nat.1) =
        ring_hom_kernel_quotient_project(f, a)
} by {
    ring_hom_kernel_quotient_project_pow(f, a, Nat.1)
    a.pow(Nat.1) = a
}

/// Quotient one raised to a natural power is quotient one for the canonical ring-kernel projection.
theorem ring_hom_kernel_quotient_project_one_pow[R: Ring, S: Ring](f: RingHom[R, S], n: Nat) {
    ring_hom_kernel_quotient_pow(f, ring_hom_kernel_quotient_one(f), n) =
        ring_hom_kernel_quotient_one(f)
} by {
    ring_hom_kernel_quotient_one_pow(f, n)
}

/// Multiplying projected ring-kernel quotient powers adds their exponents.
theorem ring_hom_kernel_quotient_project_pow_add[R: Ring, S: Ring](
    f: RingHom[R, S], a: R, n: Nat, m: Nat
) {
    ring_hom_kernel_quotient_mul(f,
        ring_hom_kernel_quotient_pow(f, ring_hom_kernel_quotient_project(f, a), n),
        ring_hom_kernel_quotient_pow(f, ring_hom_kernel_quotient_project(f, a), m)) =
    ring_hom_kernel_quotient_pow(f, ring_hom_kernel_quotient_project(f, a), n + m)
} by {
    ring_hom_kernel_quotient_pow_add(f, a, n, m)
}

/// Raising a projected ring-kernel quotient power to a power multiplies the exponents.
theorem ring_hom_kernel_quotient_project_pow_pow[R: Ring, S: Ring](
    f: RingHom[R, S], a: R, n: Nat, m: Nat
) {
    ring_hom_kernel_quotient_pow(f,
        ring_hom_kernel_quotient_pow(f, ring_hom_kernel_quotient_project(f, a), n),
        m) =
    ring_hom_kernel_quotient_pow(f, ring_hom_kernel_quotient_project(f, a), n * m)
} by {
    ring_hom_kernel_quotient_pow_pow(f, a, n, m)
}

/// Adding quotient zero on the left fixes a projected ring-kernel representative.
theorem ring_hom_kernel_quotient_project_zero_add[R: Ring, S: Ring](f: RingHom[R, S], a: R) {
    ring_hom_kernel_quotient_add(f,
        ring_hom_kernel_quotient_zero(f),
        ring_hom_kernel_quotient_project(f, a)) =
        ring_hom_kernel_quotient_project(f, a)
} by {
    ring_hom_kernel_quotient_add_zero_left_projection(f, a)
}

/// Adding quotient zero on the right fixes a projected ring-kernel representative.
theorem ring_hom_kernel_quotient_project_add_zero[R: Ring, S: Ring](f: RingHom[R, S], a: R) {
    ring_hom_kernel_quotient_add(f,
        ring_hom_kernel_quotient_project(f, a),
        ring_hom_kernel_quotient_zero(f)) =
        ring_hom_kernel_quotient_project(f, a)
} by {
    ring_hom_kernel_quotient_add_zero_right_projection(f, a)
}

/// Multiplying by quotient one on the left fixes a projected ring-kernel representative.
theorem ring_hom_kernel_quotient_project_one_mul[R: Ring, S: Ring](f: RingHom[R, S], a: R) {
    ring_hom_kernel_quotient_mul(f,
        ring_hom_kernel_quotient_one(f),
        ring_hom_kernel_quotient_project(f, a)) =
        ring_hom_kernel_quotient_project(f, a)
} by {
    ring_hom_kernel_quotient_mul_one_left_projection(f, a)
}

/// Multiplying by quotient one on the right fixes a projected ring-kernel representative.
theorem ring_hom_kernel_quotient_project_mul_one[R: Ring, S: Ring](f: RingHom[R, S], a: R) {
    ring_hom_kernel_quotient_mul(f,
        ring_hom_kernel_quotient_project(f, a),
        ring_hom_kernel_quotient_one(f)) =
        ring_hom_kernel_quotient_project(f, a)
} by {
    ring_hom_kernel_quotient_mul_one_right_projection(f, a)
}

/// Multiplying by quotient zero on the left gives quotient zero.
theorem ring_hom_kernel_quotient_project_zero_mul[R: Ring, S: Ring](f: RingHom[R, S], a: R) {
    ring_hom_kernel_quotient_mul(f,
        ring_hom_kernel_quotient_zero(f),
        ring_hom_kernel_quotient_project(f, a)) =
        ring_hom_kernel_quotient_zero(f)
} by {
    ring_hom_kernel_quotient_mul_zero_right(f, a)
    ring_hom_kernel_quotient_zero_projection(f)
}

/// Multiplying by quotient zero on the right gives quotient zero.
theorem ring_hom_kernel_quotient_project_mul_zero[R: Ring, S: Ring](f: RingHom[R, S], a: R) {
    ring_hom_kernel_quotient_mul(f,
        ring_hom_kernel_quotient_project(f, a),
        ring_hom_kernel_quotient_zero(f)) =
        ring_hom_kernel_quotient_zero(f)
} by {
    ring_hom_kernel_quotient_mul_zero_left(f, a)
    ring_hom_kernel_quotient_zero_projection(f)
}

/// Addition of projected ring-kernel representatives is associative.
theorem ring_hom_kernel_quotient_project_add_assoc[R: Ring, S: Ring](f: RingHom[R, S], a: R, b: R, c: R) {
    ring_hom_kernel_quotient_add(f,
        ring_hom_kernel_quotient_add(f,
            ring_hom_kernel_quotient_project(f, a),
            ring_hom_kernel_quotient_project(f, b)),
        ring_hom_kernel_quotient_project(f, c)) =
    ring_hom_kernel_quotient_add(f,
        ring_hom_kernel_quotient_project(f, a),
        ring_hom_kernel_quotient_add(f,
            ring_hom_kernel_quotient_project(f, b),
            ring_hom_kernel_quotient_project(f, c)))
} by {
    ring_hom_kernel_quotient_add_assoc(f, a, b, c)
}

/// Multiplication of projected ring-kernel representatives is associative.
theorem ring_hom_kernel_quotient_project_mul_assoc[R: Ring, S: Ring](f: RingHom[R, S], a: R, b: R, c: R) {
    ring_hom_kernel_quotient_mul(f,
        ring_hom_kernel_quotient_mul(f,
            ring_hom_kernel_quotient_project(f, a),
            ring_hom_kernel_quotient_project(f, b)),
        ring_hom_kernel_quotient_project(f, c)) =
    ring_hom_kernel_quotient_mul(f,
        ring_hom_kernel_quotient_project(f, a),
        ring_hom_kernel_quotient_mul(f,
            ring_hom_kernel_quotient_project(f, b),
            ring_hom_kernel_quotient_project(f, c)))
} by {
    ring_hom_kernel_quotient_mul_assoc(f, a, b, c)
}

/// Addition of projected ring-kernel representatives is commutative.
theorem ring_hom_kernel_quotient_project_add_comm[R: Ring, S: Ring](f: RingHom[R, S], a: R, b: R) {
    ring_hom_kernel_quotient_add(f,
        ring_hom_kernel_quotient_project(f, a),
        ring_hom_kernel_quotient_project(f, b)) =
    ring_hom_kernel_quotient_add(f,
        ring_hom_kernel_quotient_project(f, b),
        ring_hom_kernel_quotient_project(f, a))
} by {
    ring_hom_kernel_quotient_add_comm(f, a, b)
}

/// Multiplication of projected ring-kernel representatives is commutative in the commutative case.
theorem ring_hom_kernel_quotient_project_mul_comm[R: CommRing, S: CommRing](f: RingHom[R, S], a: R, b: R) {
    ring_hom_kernel_quotient_mul(f,
        ring_hom_kernel_quotient_project(f, a),
        ring_hom_kernel_quotient_project(f, b)) =
    ring_hom_kernel_quotient_mul(f,
        ring_hom_kernel_quotient_project(f, b),
        ring_hom_kernel_quotient_project(f, a))
} by {
    ring_hom_kernel_quotient_mul_comm(f, a, b)
}

/// The quotient negation is a left inverse for projected ring-kernel addition.
theorem ring_hom_kernel_quotient_project_neg_add[R: Ring, S: Ring](f: RingHom[R, S], a: R) {
    ring_hom_kernel_quotient_add(f,
        ring_hom_kernel_quotient_neg(f, ring_hom_kernel_quotient_project(f, a)),
        ring_hom_kernel_quotient_project(f, a)) =
        ring_hom_kernel_quotient_zero(f)
} by {
    ring_hom_kernel_quotient_neg_add(f, a)
    ring_hom_kernel_quotient_zero_projection(f)
}

/// The quotient negation is a right inverse for projected ring-kernel addition.
theorem ring_hom_kernel_quotient_project_add_neg[R: Ring, S: Ring](f: RingHom[R, S], a: R) {
    ring_hom_kernel_quotient_add(f,
        ring_hom_kernel_quotient_project(f, a),
        ring_hom_kernel_quotient_neg(f, ring_hom_kernel_quotient_project(f, a))) =
        ring_hom_kernel_quotient_zero(f)
} by {
    ring_hom_kernel_quotient_add_neg(f, a)
    ring_hom_kernel_quotient_zero_projection(f)
}

/// Subtracting quotient zero fixes a projected ring-kernel representative.
theorem ring_hom_kernel_quotient_project_sub_zero[R: Ring, S: Ring](f: RingHom[R, S], a: R) {
    ring_hom_kernel_quotient_sub(f,
        ring_hom_kernel_quotient_project(f, a),
        ring_hom_kernel_quotient_zero(f)) =
        ring_hom_kernel_quotient_project(f, a)
} by {
    ring_hom_kernel_quotient_sub_zero(f, a)
    ring_hom_kernel_quotient_zero_projection(f)
}

/// Subtracting a projected ring-kernel representative from zero gives quotient negation.
theorem ring_hom_kernel_quotient_project_zero_sub[R: Ring, S: Ring](f: RingHom[R, S], a: R) {
    ring_hom_kernel_quotient_sub(f,
        ring_hom_kernel_quotient_zero(f),
        ring_hom_kernel_quotient_project(f, a)) =
        ring_hom_kernel_quotient_neg(f, ring_hom_kernel_quotient_project(f, a))
} by {
    ring_hom_kernel_quotient_zero_sub(f, a)
    ring_hom_kernel_quotient_zero_projection(f)
}

/// Subtracting a projected ring-kernel representative from itself gives quotient zero.
theorem ring_hom_kernel_quotient_project_sub_self[R: Ring, S: Ring](f: RingHom[R, S], a: R) {
    ring_hom_kernel_quotient_sub(f,
        ring_hom_kernel_quotient_project(f, a),
        ring_hom_kernel_quotient_project(f, a)) =
        ring_hom_kernel_quotient_zero(f)
} by {
    ring_hom_kernel_quotient_sub_self(f, a)
    ring_hom_kernel_quotient_zero_projection(f)
}

/// Projected ring-kernel quotient subtraction is addition of quotient negation.
theorem ring_hom_kernel_quotient_project_sub_eq_add_neg[R: Ring, S: Ring](f: RingHom[R, S], a: R, b: R) {
    ring_hom_kernel_quotient_sub(f,
        ring_hom_kernel_quotient_project(f, a),
        ring_hom_kernel_quotient_project(f, b)) =
    ring_hom_kernel_quotient_add(f,
        ring_hom_kernel_quotient_project(f, a),
        ring_hom_kernel_quotient_neg(f, ring_hom_kernel_quotient_project(f, b)))
} by {
    ring_hom_kernel_quotient_sub_eq_add_neg(f, a, b)
}

/// A projected ring-kernel quotient negation cancels a matching left summand.
theorem ring_hom_kernel_quotient_project_neg_add_cancel_left[R: Ring, S: Ring](
    f: RingHom[R, S], a: R, b: R
) {
    ring_hom_kernel_quotient_add(f,
        ring_hom_kernel_quotient_neg(f,
            ring_hom_kernel_quotient_project(f, a)),
        ring_hom_kernel_quotient_add(f,
            ring_hom_kernel_quotient_project(f, a),
            ring_hom_kernel_quotient_project(f, b))) =
        ring_hom_kernel_quotient_project(f, b)
} by {
    ring_hom_kernel_quotient_neg_add_cancel_left(f, a, b)
}

/// A projected ring-kernel quotient negation cancels a matching right summand.
theorem ring_hom_kernel_quotient_project_add_neg_cancel_right[R: Ring, S: Ring](
    f: RingHom[R, S], a: R, b: R
) {
    ring_hom_kernel_quotient_add(f,
        ring_hom_kernel_quotient_add(f,
            ring_hom_kernel_quotient_project(f, b),
            ring_hom_kernel_quotient_project(f, a)),
        ring_hom_kernel_quotient_neg(f,
            ring_hom_kernel_quotient_project(f, a))) =
        ring_hom_kernel_quotient_project(f, b)
} by {
    ring_hom_kernel_quotient_add_neg_cancel_right(f, a, b)
}

/// Subtracting the right summand of a projected ring-kernel quotient sum cancels it.
theorem ring_hom_kernel_quotient_project_add_sub_cancel_right[R: Ring, S: Ring](
    f: RingHom[R, S], a: R, b: R
) {
    ring_hom_kernel_quotient_sub(f,
        ring_hom_kernel_quotient_add(f,
            ring_hom_kernel_quotient_project(f, a),
            ring_hom_kernel_quotient_project(f, b)),
        ring_hom_kernel_quotient_project(f, b)) =
        ring_hom_kernel_quotient_project(f, a)
} by {
    ring_hom_kernel_quotient_add_sub_cancel_right(f, a, b)
}

/// Subtracting the left summand of a projected ring-kernel quotient sum cancels it.
theorem ring_hom_kernel_quotient_project_add_sub_cancel_left[R: Ring, S: Ring](
    f: RingHom[R, S], a: R, b: R
) {
    ring_hom_kernel_quotient_sub(f,
        ring_hom_kernel_quotient_add(f,
            ring_hom_kernel_quotient_project(f, a),
            ring_hom_kernel_quotient_project(f, b)),
        ring_hom_kernel_quotient_project(f, a)) =
        ring_hom_kernel_quotient_project(f, b)
} by {
    ring_hom_kernel_quotient_add_sub_cancel_left(f, a, b)
}

/// Ring-kernel quotient negation sends quotient zero to quotient zero.
theorem ring_hom_kernel_quotient_project_neg_zero[R: Ring, S: Ring](f: RingHom[R, S]) {
    ring_hom_kernel_quotient_neg(f, ring_hom_kernel_quotient_zero(f)) =
        ring_hom_kernel_quotient_zero(f)
} by {
    ring_hom_kernel_quotient_neg_zero(f)
}

/// Ring-kernel quotient negation is involutive on projected representatives.
theorem ring_hom_kernel_quotient_project_neg_neg[R: Ring, S: Ring](f: RingHom[R, S], a: R) {
    ring_hom_kernel_quotient_neg(f,
        ring_hom_kernel_quotient_neg(f, ring_hom_kernel_quotient_project(f, a))) =
        ring_hom_kernel_quotient_project(f, a)
} by {
    ring_hom_kernel_quotient_neg_neg(f, a)
}

/// The canonical projection to a monoid-kernel quotient.
define monoid_hom_kernel_quotient_project[M: Monoid, N: Monoid](f: MonoidHom[M, N], a: M) -> QuotientOver[M] {
    quotient_over_mk(kernel_quotient_relation(f.hom), a)
}

/// The canonical monoid-kernel projection is the quotient constructor.
theorem monoid_hom_kernel_quotient_project_eq_mk[M: Monoid, N: Monoid](f: MonoidHom[M, N], a: M) {
    monoid_hom_kernel_quotient_project(f, a) = quotient_over_mk(kernel_quotient_relation(f.hom), a)
}

/// Equality of projected monoid-kernel representatives is the kernel relation.
theorem monoid_hom_kernel_quotient_project_eq_iff_kernel_relation[M: Monoid, N: Monoid](
    f: MonoidHom[M, N], a: M, b: M
) {
    (monoid_hom_kernel_quotient_project(f, a) = monoid_hom_kernel_quotient_project(f, b)) =
        kernel_relation(f.hom, a, b)
} by {
    monoid_hom_kernel_quotient_project_eq_mk(f, a)
    monoid_hom_kernel_quotient_project_eq_mk(f, b)
    quotient_over_mk_eq_iff_rel(kernel_quotient_relation(f.hom), a, b)
    kernel_quotient_relation_rel(f.hom)
}

/// The homomorphism out of the monoid-kernel quotient induced by `f`.
define monoid_hom_kernel_quotient_induced[M: Monoid, N: Monoid](f: MonoidHom[M, N]) ->
    QuotientOver[M] -> N {
    quotient_over_lift_to(f.hom)
}

/// The induced map agrees with `f` on canonical monoid-kernel projections.
theorem monoid_hom_kernel_quotient_induced_project[M: Monoid, N: Monoid](f: MonoidHom[M, N], a: M) {
    monoid_hom_kernel_quotient_induced(f, monoid_hom_kernel_quotient_project(f, a)) = f.hom(a)
} by {
    kernel_relation_respects(f.hom)
    monoid_hom_kernel_quotient_project_eq_mk(f, a)
    quotient_over_lift_to_projection(kernel_quotient_relation(f.hom), f.hom, a)
}

/// The induced map out of the monoid-kernel quotient separates projections of unrelated representatives.
theorem monoid_hom_kernel_quotient_induced_injective[M: Monoid, N: Monoid](
    f: MonoidHom[M, N], a: M, b: M
) {
    monoid_hom_kernel_quotient_induced(f, monoid_hom_kernel_quotient_project(f, a)) =
        monoid_hom_kernel_quotient_induced(f, monoid_hom_kernel_quotient_project(f, b))
    implies monoid_hom_kernel_quotient_project(f, a) = monoid_hom_kernel_quotient_project(f, b)
} by {
    if monoid_hom_kernel_quotient_induced(f, monoid_hom_kernel_quotient_project(f, a)) =
        monoid_hom_kernel_quotient_induced(f, monoid_hom_kernel_quotient_project(f, b)) {
        monoid_hom_kernel_quotient_induced_project(f, a)
        monoid_hom_kernel_quotient_induced_project(f, b)
        f.hom(a) = f.hom(b)
        kernel_relation(f.hom, a, b) = (f.hom(a) = f.hom(b))
        kernel_relation(f.hom, a, b)
        monoid_hom_kernel_quotient_project_eq_iff_kernel_relation(f, a, b)
        monoid_hom_kernel_quotient_project(f, a) = monoid_hom_kernel_quotient_project(f, b)
    }
}

/// The canonical monoid-kernel projection preserves the identity.
theorem monoid_hom_kernel_quotient_project_one[M: Monoid, N: Monoid](f: MonoidHom[M, N]) {
    monoid_hom_kernel_quotient_project(f, M.1) = monoid_hom_kernel_quotient_one(f)
} by {
    monoid_hom_kernel_quotient_one_projection(f)
}

/// The canonical monoid-kernel projection preserves multiplication.
theorem monoid_hom_kernel_quotient_project_mul[M: Monoid, N: Monoid](f: MonoidHom[M, N], a: M, b: M) {
    monoid_hom_kernel_quotient_project(f, a * b) =
    monoid_hom_kernel_quotient_mul(f,
        monoid_hom_kernel_quotient_project(f, a),
        monoid_hom_kernel_quotient_project(f, b))
} by {
    monoid_hom_kernel_quotient_mul_projection(f, a, b)
    monoid_hom_kernel_quotient_project(f, a * b) =
        quotient_over_mk(kernel_quotient_relation(f.hom), a * b)
    monoid_hom_kernel_quotient_project(f, a) =
        quotient_over_mk(kernel_quotient_relation(f.hom), a)
    monoid_hom_kernel_quotient_project(f, b) =
        quotient_over_mk(kernel_quotient_relation(f.hom), b)
}

/// The induced map out of the monoid-kernel quotient preserves multiplication.
theorem monoid_hom_kernel_quotient_induced_mul[M: Monoid, N: Monoid](f: MonoidHom[M, N], a: M, b: M) {
    monoid_hom_kernel_quotient_induced(f,
        monoid_hom_kernel_quotient_mul(f,
            monoid_hom_kernel_quotient_project(f, a),
            monoid_hom_kernel_quotient_project(f, b))) =
        monoid_hom_kernel_quotient_induced(f, monoid_hom_kernel_quotient_project(f, a)) *
        monoid_hom_kernel_quotient_induced(f, monoid_hom_kernel_quotient_project(f, b))
} by {
    monoid_hom_kernel_quotient_project_mul(f, a, b)
    monoid_hom_kernel_quotient_induced_project(f, a * b)
    monoid_hom_kernel_quotient_induced_project(f, a)
    monoid_hom_kernel_quotient_induced_project(f, b)
    monoid_hom_mul(f, a, b)
}

/// The induced map out of the monoid-kernel quotient preserves the identity.
theorem monoid_hom_kernel_quotient_induced_one[M: Monoid, N: Monoid](f: MonoidHom[M, N]) {
    monoid_hom_kernel_quotient_induced(f, monoid_hom_kernel_quotient_one(f)) = N.1
} by {
    monoid_hom_kernel_quotient_project_one(f)
    monoid_hom_kernel_quotient_induced_project(f, M.1)
    monoid_hom_one(f)
}

/// Every value of `f` is attained by the induced map out of the monoid-kernel quotient.
theorem monoid_hom_kernel_quotient_induced_surjective_onto_image[M: Monoid, N: Monoid](
    f: MonoidHom[M, N], a: M
) {
    exists(q: QuotientOver[M]) {
        monoid_hom_kernel_quotient_induced(f, q) = f.hom(a)
    }
} by {
    monoid_hom_kernel_quotient_induced_project(f, a)
    monoid_hom_kernel_quotient_induced(f, monoid_hom_kernel_quotient_project(f, a)) = f.hom(a)
}

/// Multiplication of canonical monoid-kernel projections respects related left representatives.
theorem monoid_hom_kernel_quotient_project_mul_left_of_kernel_relation[M: Monoid, N: Monoid](
    f: MonoidHom[M, N], a1: M, a2: M, b: M
) {
    kernel_relation(f.hom, a1, a2) implies
    monoid_hom_kernel_quotient_mul(f,
        monoid_hom_kernel_quotient_project(f, a1),
        monoid_hom_kernel_quotient_project(f, b)) =
    monoid_hom_kernel_quotient_mul(f,
        monoid_hom_kernel_quotient_project(f, a2),
        monoid_hom_kernel_quotient_project(f, b))
} by {
    if kernel_relation(f.hom, a1, a2) {
        monoid_hom_kernel_quotient_mul_projection_left(f, a1, a2, b)
    }
}

/// Multiplication of canonical monoid-kernel projections respects related right representatives.
theorem monoid_hom_kernel_quotient_project_mul_right_of_kernel_relation[M: Monoid, N: Monoid](
    f: MonoidHom[M, N], a: M, b1: M, b2: M
) {
    kernel_relation(f.hom, b1, b2) implies
    monoid_hom_kernel_quotient_mul(f,
        monoid_hom_kernel_quotient_project(f, a),
        monoid_hom_kernel_quotient_project(f, b1)) =
    monoid_hom_kernel_quotient_mul(f,
        monoid_hom_kernel_quotient_project(f, a),
        monoid_hom_kernel_quotient_project(f, b2))
} by {
    if kernel_relation(f.hom, b1, b2) {
        monoid_hom_kernel_quotient_mul_projection_right(f, a, b1, b2)
    }
}

/// The zeroth power of a projected monoid-kernel representative is the projected identity.
theorem monoid_hom_kernel_quotient_project_pow_zero[M: Monoid, N: Monoid](f: MonoidHom[M, N], a: M) {
    monoid_hom_kernel_quotient_pow(f, monoid_hom_kernel_quotient_project(f, a), Nat.0) =
        monoid_hom_kernel_quotient_project(f, a.pow(Nat.0))
} by {
    monoid_hom_kernel_quotient_pow_mk_zero(f, a)
}

/// Powers of projected monoid-kernel representatives agree with projected powers.
theorem monoid_hom_kernel_quotient_project_pow[M: Monoid, N: Monoid](f: MonoidHom[M, N], a: M, n: Nat) {
    monoid_hom_kernel_quotient_pow(f, monoid_hom_kernel_quotient_project(f, a), n) =
        monoid_hom_kernel_quotient_project(f, a.pow(n))
} by {
    monoid_hom_kernel_quotient_pow_mk(f, a, n)
}

/// The first power of a projected monoid-kernel representative is the projected representative.
theorem monoid_hom_kernel_quotient_project_pow_one[M: Monoid, N: Monoid](f: MonoidHom[M, N], a: M) {
    monoid_hom_kernel_quotient_pow(f, monoid_hom_kernel_quotient_project(f, a), Nat.1) =
        monoid_hom_kernel_quotient_project(f, a)
} by {
    monoid_hom_kernel_quotient_project_pow(f, a, Nat.1)
    a.pow(Nat.1) = a
}

/// Quotient one raised to a natural power is quotient one in a projected monoid-kernel quotient.
theorem monoid_hom_kernel_quotient_project_one_pow[M: Monoid, N: Monoid](f: MonoidHom[M, N], n: Nat) {
    monoid_hom_kernel_quotient_pow(f, monoid_hom_kernel_quotient_one(f), n) =
        monoid_hom_kernel_quotient_one(f)
} by {
    monoid_hom_kernel_quotient_one_pow(f, n)
}

/// Multiplying projected monoid-kernel quotient powers adds their exponents.
theorem monoid_hom_kernel_quotient_project_pow_add[M: Monoid, N: Monoid](
    f: MonoidHom[M, N], a: M, n: Nat, m: Nat
) {
    monoid_hom_kernel_quotient_mul(f,
        monoid_hom_kernel_quotient_pow(f, monoid_hom_kernel_quotient_project(f, a), n),
        monoid_hom_kernel_quotient_pow(f, monoid_hom_kernel_quotient_project(f, a), m)) =
    monoid_hom_kernel_quotient_pow(f, monoid_hom_kernel_quotient_project(f, a), n + m)
} by {
    monoid_hom_kernel_quotient_pow_add(f, a, n, m)
}

/// Raising a projected monoid-kernel quotient power to a power multiplies the exponents.
theorem monoid_hom_kernel_quotient_project_pow_pow[M: Monoid, N: Monoid](
    f: MonoidHom[M, N], a: M, n: Nat, m: Nat
) {
    monoid_hom_kernel_quotient_pow(f,
        monoid_hom_kernel_quotient_pow(f, monoid_hom_kernel_quotient_project(f, a), n),
        m) =
    monoid_hom_kernel_quotient_pow(f, monoid_hom_kernel_quotient_project(f, a), n * m)
} by {
    monoid_hom_kernel_quotient_pow_pow(f, a, n, m)
}

/// Multiplication of projected monoid-kernel representatives is associative.
theorem monoid_hom_kernel_quotient_project_mul_assoc[M: Monoid, N: Monoid](f: MonoidHom[M, N], a: M, b: M, c: M) {
    monoid_hom_kernel_quotient_mul(f,
        monoid_hom_kernel_quotient_mul(f,
            monoid_hom_kernel_quotient_project(f, a),
            monoid_hom_kernel_quotient_project(f, b)),
        monoid_hom_kernel_quotient_project(f, c)) =
    monoid_hom_kernel_quotient_mul(f,
        monoid_hom_kernel_quotient_project(f, a),
        monoid_hom_kernel_quotient_mul(f,
            monoid_hom_kernel_quotient_project(f, b),
            monoid_hom_kernel_quotient_project(f, c)))
} by {
    monoid_hom_kernel_quotient_mul_assoc(f, a, b, c)
}

/// The quotient identity is a left identity for projected monoid-kernel representatives.
theorem monoid_hom_kernel_quotient_project_one_mul[M: Monoid, N: Monoid](f: MonoidHom[M, N], a: M) {
    monoid_hom_kernel_quotient_mul(f,
        monoid_hom_kernel_quotient_one(f),
        monoid_hom_kernel_quotient_project(f, a)) =
        monoid_hom_kernel_quotient_project(f, a)
} by {
    monoid_hom_kernel_quotient_mul_one_left_projection(f, a)
}

/// The quotient identity is a right identity for projected monoid-kernel representatives.
theorem monoid_hom_kernel_quotient_project_mul_one[M: Monoid, N: Monoid](f: MonoidHom[M, N], a: M) {
    monoid_hom_kernel_quotient_mul(f,
        monoid_hom_kernel_quotient_project(f, a),
        monoid_hom_kernel_quotient_one(f)) =
        monoid_hom_kernel_quotient_project(f, a)
} by {
    monoid_hom_kernel_quotient_mul_one_right_projection(f, a)
}

/// Multiplication of projected monoid-kernel representatives is commutative in the commutative case.
theorem monoid_hom_kernel_quotient_project_mul_comm[M: CommMonoid, N: CommMonoid](
    f: MonoidHom[M, N], a: M, b: M
) {
    monoid_hom_kernel_quotient_mul(f,
        monoid_hom_kernel_quotient_project(f, a),
        monoid_hom_kernel_quotient_project(f, b)) =
    monoid_hom_kernel_quotient_mul(f,
        monoid_hom_kernel_quotient_project(f, b),
        monoid_hom_kernel_quotient_project(f, a))
} by {
    monoid_hom_kernel_quotient_mul_comm(f, a, b)
}

/// The canonical projection to an additive-monoid-kernel quotient.
define add_monoid_hom_kernel_quotient_project[A: AddMonoid, B: AddMonoid](
    f: AddMonoidHom[A, B], a: A
) -> QuotientOver[A] {
    quotient_over_mk(kernel_quotient_relation(f.hom), a)
}

/// The canonical additive-monoid-kernel projection is the quotient constructor.
theorem add_monoid_hom_kernel_quotient_project_eq_mk[A: AddMonoid, B: AddMonoid](
    f: AddMonoidHom[A, B], a: A
) {
    add_monoid_hom_kernel_quotient_project(f, a) = quotient_over_mk(kernel_quotient_relation(f.hom), a)
}

/// Equality of projected additive-monoid-kernel representatives is the kernel relation.
theorem add_monoid_hom_kernel_quotient_project_eq_iff_kernel_relation[A: AddMonoid, B: AddMonoid](
    f: AddMonoidHom[A, B], a: A, b: A
) {
    (add_monoid_hom_kernel_quotient_project(f, a) = add_monoid_hom_kernel_quotient_project(f, b)) =
        kernel_relation(f.hom, a, b)
} by {
    add_monoid_hom_kernel_quotient_project_eq_mk(f, a)
    add_monoid_hom_kernel_quotient_project_eq_mk(f, b)
    quotient_over_mk_eq_iff_rel(kernel_quotient_relation(f.hom), a, b)
    kernel_quotient_relation_rel(f.hom)
}

/// The homomorphism out of the additive-monoid-kernel quotient induced by `f`.
define add_monoid_hom_kernel_quotient_induced[A: AddMonoid, B: AddMonoid](f: AddMonoidHom[A, B]) ->
    QuotientOver[A] -> B {
    quotient_over_lift_to(f.hom)
}

/// The induced map agrees with `f` on canonical additive-monoid-kernel projections.
theorem add_monoid_hom_kernel_quotient_induced_project[A: AddMonoid, B: AddMonoid](
    f: AddMonoidHom[A, B], a: A
) {
    add_monoid_hom_kernel_quotient_induced(f, add_monoid_hom_kernel_quotient_project(f, a)) = f.hom(a)
} by {
    kernel_relation_respects(f.hom)
    add_monoid_hom_kernel_quotient_project_eq_mk(f, a)
    quotient_over_lift_to_projection(kernel_quotient_relation(f.hom), f.hom, a)
}

/// The induced map out of the additive-monoid-kernel quotient separates projections of unrelated representatives.
theorem add_monoid_hom_kernel_quotient_induced_injective[A: AddMonoid, B: AddMonoid](
    f: AddMonoidHom[A, B], a: A, b: A
) {
    add_monoid_hom_kernel_quotient_induced(f, add_monoid_hom_kernel_quotient_project(f, a)) =
        add_monoid_hom_kernel_quotient_induced(f, add_monoid_hom_kernel_quotient_project(f, b))
    implies add_monoid_hom_kernel_quotient_project(f, a) = add_monoid_hom_kernel_quotient_project(f, b)
} by {
    if add_monoid_hom_kernel_quotient_induced(f, add_monoid_hom_kernel_quotient_project(f, a)) =
        add_monoid_hom_kernel_quotient_induced(f, add_monoid_hom_kernel_quotient_project(f, b)) {
        add_monoid_hom_kernel_quotient_induced_project(f, a)
        add_monoid_hom_kernel_quotient_induced_project(f, b)
        f.hom(a) = f.hom(b)
        kernel_relation(f.hom, a, b) = (f.hom(a) = f.hom(b))
        kernel_relation(f.hom, a, b)
        add_monoid_hom_kernel_quotient_project_eq_iff_kernel_relation(f, a, b)
        add_monoid_hom_kernel_quotient_project(f, a) = add_monoid_hom_kernel_quotient_project(f, b)
    }
}

/// The canonical additive-monoid-kernel projection preserves zero.
theorem add_monoid_hom_kernel_quotient_project_zero[A: AddMonoid, B: AddMonoid](
    f: AddMonoidHom[A, B]
) {
    add_monoid_hom_kernel_quotient_project(f, A.0) = add_monoid_hom_kernel_quotient_zero(f)
} by {
    add_monoid_hom_kernel_quotient_zero_projection(f)
}

/// The canonical additive-monoid-kernel projection preserves addition.
theorem add_monoid_hom_kernel_quotient_project_add[A: AddMonoid, B: AddMonoid](
    f: AddMonoidHom[A, B], a: A, b: A
) {
    add_monoid_hom_kernel_quotient_project(f, a + b) =
    add_monoid_hom_kernel_quotient_add(f,
        add_monoid_hom_kernel_quotient_project(f, a),
        add_monoid_hom_kernel_quotient_project(f, b))
} by {
    add_monoid_hom_kernel_quotient_add_projection(f, a, b)
    add_monoid_hom_kernel_quotient_project(f, a + b) =
        quotient_over_mk(kernel_quotient_relation(f.hom), a + b)
    add_monoid_hom_kernel_quotient_project(f, a) =
        quotient_over_mk(kernel_quotient_relation(f.hom), a)
    add_monoid_hom_kernel_quotient_project(f, b) =
        quotient_over_mk(kernel_quotient_relation(f.hom), b)
}

/// The induced map out of the additive-monoid-kernel quotient preserves addition.
theorem add_monoid_hom_kernel_quotient_induced_add[A: AddMonoid, B: AddMonoid](
    f: AddMonoidHom[A, B], a: A, b: A
) {
    add_monoid_hom_kernel_quotient_induced(f,
        add_monoid_hom_kernel_quotient_add(f,
            add_monoid_hom_kernel_quotient_project(f, a),
            add_monoid_hom_kernel_quotient_project(f, b))) =
        add_monoid_hom_kernel_quotient_induced(f, add_monoid_hom_kernel_quotient_project(f, a)) +
        add_monoid_hom_kernel_quotient_induced(f, add_monoid_hom_kernel_quotient_project(f, b))
} by {
    add_monoid_hom_kernel_quotient_project_add(f, a, b)
    add_monoid_hom_kernel_quotient_induced_project(f, a + b)
    add_monoid_hom_kernel_quotient_induced_project(f, a)
    add_monoid_hom_kernel_quotient_induced_project(f, b)
    add_monoid_hom_add(f, a, b)
}

/// The induced map out of the additive-monoid-kernel quotient preserves zero.
theorem add_monoid_hom_kernel_quotient_induced_zero[A: AddMonoid, B: AddMonoid](f: AddMonoidHom[A, B]) {
    add_monoid_hom_kernel_quotient_induced(f, add_monoid_hom_kernel_quotient_zero(f)) = B.0
} by {
    add_monoid_hom_kernel_quotient_project_zero(f)
    add_monoid_hom_kernel_quotient_induced_project(f, A.0)
    add_monoid_hom_zero(f)
}

/// Every value of `f` is attained by the induced map out of the additive-monoid-kernel quotient.
theorem add_monoid_hom_kernel_quotient_induced_surjective_onto_image[A: AddMonoid, B: AddMonoid](
    f: AddMonoidHom[A, B], a: A
) {
    exists(q: QuotientOver[A]) {
        add_monoid_hom_kernel_quotient_induced(f, q) = f.hom(a)
    }
} by {
    add_monoid_hom_kernel_quotient_induced_project(f, a)
    add_monoid_hom_kernel_quotient_induced(f, add_monoid_hom_kernel_quotient_project(f, a)) = f.hom(a)
}

/// Addition of canonical additive-monoid-kernel projections respects related left representatives.
theorem add_monoid_hom_kernel_quotient_project_add_left_of_kernel_relation[A: AddMonoid, B: AddMonoid](
    f: AddMonoidHom[A, B], a1: A, a2: A, b: A
) {
    kernel_relation(f.hom, a1, a2) implies
    add_monoid_hom_kernel_quotient_add(f,
        add_monoid_hom_kernel_quotient_project(f, a1),
        add_monoid_hom_kernel_quotient_project(f, b)) =
    add_monoid_hom_kernel_quotient_add(f,
        add_monoid_hom_kernel_quotient_project(f, a2),
        add_monoid_hom_kernel_quotient_project(f, b))
} by {
    if kernel_relation(f.hom, a1, a2) {
        add_monoid_hom_kernel_quotient_add_projection_left(f, a1, a2, b)
    }
}

/// Addition of canonical additive-monoid-kernel projections respects related right representatives.
theorem add_monoid_hom_kernel_quotient_project_add_right_of_kernel_relation[A: AddMonoid, B: AddMonoid](
    f: AddMonoidHom[A, B], a: A, b1: A, b2: A
) {
    kernel_relation(f.hom, b1, b2) implies
    add_monoid_hom_kernel_quotient_add(f,
        add_monoid_hom_kernel_quotient_project(f, a),
        add_monoid_hom_kernel_quotient_project(f, b1)) =
    add_monoid_hom_kernel_quotient_add(f,
        add_monoid_hom_kernel_quotient_project(f, a),
        add_monoid_hom_kernel_quotient_project(f, b2))
} by {
    if kernel_relation(f.hom, b1, b2) {
        add_monoid_hom_kernel_quotient_add_projection_right(f, a, b1, b2)
    }
}


/// Addition of projected additive-monoid-kernel representatives is associative.
theorem add_monoid_hom_kernel_quotient_project_add_assoc[A: AddMonoid, B: AddMonoid](
    f: AddMonoidHom[A, B], a: A, b: A, c: A
) {
    add_monoid_hom_kernel_quotient_add(f,
        add_monoid_hom_kernel_quotient_add(f,
            add_monoid_hom_kernel_quotient_project(f, a),
            add_monoid_hom_kernel_quotient_project(f, b)),
        add_monoid_hom_kernel_quotient_project(f, c)) =
    add_monoid_hom_kernel_quotient_add(f,
        add_monoid_hom_kernel_quotient_project(f, a),
        add_monoid_hom_kernel_quotient_add(f,
            add_monoid_hom_kernel_quotient_project(f, b),
            add_monoid_hom_kernel_quotient_project(f, c)))
} by {
    add_monoid_hom_kernel_quotient_add_assoc(f, a, b, c)
}

/// The quotient zero is a left identity for projected additive-monoid-kernel representatives.
theorem add_monoid_hom_kernel_quotient_project_zero_add[A: AddMonoid, B: AddMonoid](
    f: AddMonoidHom[A, B], a: A
) {
    add_monoid_hom_kernel_quotient_add(f,
        add_monoid_hom_kernel_quotient_zero(f),
        add_monoid_hom_kernel_quotient_project(f, a)) =
        add_monoid_hom_kernel_quotient_project(f, a)
} by {
    add_monoid_hom_kernel_quotient_add_zero_left_projection(f, a)
}

/// The quotient zero is a right identity for projected additive-monoid-kernel representatives.
theorem add_monoid_hom_kernel_quotient_project_add_zero[A: AddMonoid, B: AddMonoid](
    f: AddMonoidHom[A, B], a: A
) {
    add_monoid_hom_kernel_quotient_add(f,
        add_monoid_hom_kernel_quotient_project(f, a),
        add_monoid_hom_kernel_quotient_zero(f)) =
        add_monoid_hom_kernel_quotient_project(f, a)
} by {
    add_monoid_hom_kernel_quotient_add_zero_right_projection(f, a)
}

/// Addition of projected additive-monoid-kernel representatives is commutative in the commutative case.
theorem add_monoid_hom_kernel_quotient_project_add_comm[A: AddCommMonoid, B: AddCommMonoid](
    f: AddMonoidHom[A, B], a: A, b: A
) {
    add_monoid_hom_kernel_quotient_add(f,
        add_monoid_hom_kernel_quotient_project(f, a),
        add_monoid_hom_kernel_quotient_project(f, b)) =
    add_monoid_hom_kernel_quotient_add(f,
        add_monoid_hom_kernel_quotient_project(f, b),
        add_monoid_hom_kernel_quotient_project(f, a))
} by {
    add_monoid_hom_kernel_quotient_add_comm(f, a, b)
}

/// The canonical projection to an additive-group-kernel quotient.
define add_group_hom_kernel_quotient_project[A: AddGroup, B: AddGroup](f: AddGroupHom[A, B], a: A) -> QuotientOver[A] {
    quotient_over_mk(kernel_quotient_relation(f.hom), a)
}

/// The canonical additive-group-kernel projection is the quotient constructor.
theorem add_group_hom_kernel_quotient_project_eq_mk[A: AddGroup, B: AddGroup](f: AddGroupHom[A, B], a: A) {
    add_group_hom_kernel_quotient_project(f, a) = quotient_over_mk(kernel_quotient_relation(f.hom), a)
}

/// Equality of projected additive-group-kernel representatives is the kernel relation.
theorem add_group_hom_kernel_quotient_project_eq_iff_kernel_relation[A: AddGroup, B: AddGroup](
    f: AddGroupHom[A, B], a: A, b: A
) {
    (add_group_hom_kernel_quotient_project(f, a) = add_group_hom_kernel_quotient_project(f, b)) =
        kernel_relation(f.hom, a, b)
} by {
    add_group_hom_kernel_quotient_project_eq_mk(f, a)
    add_group_hom_kernel_quotient_project_eq_mk(f, b)
    quotient_over_mk_eq_iff_rel(kernel_quotient_relation(f.hom), a, b)
    kernel_quotient_relation_rel(f.hom)
}

/// The homomorphism out of the additive-group-kernel quotient induced by `f`.
define add_group_hom_kernel_quotient_induced[A: AddGroup, B: AddGroup](f: AddGroupHom[A, B]) ->
    QuotientOver[A] -> B {
    quotient_over_lift_to(f.hom)
}

/// The induced map agrees with `f` on canonical additive-group-kernel projections.
theorem add_group_hom_kernel_quotient_induced_project[A: AddGroup, B: AddGroup](
    f: AddGroupHom[A, B], a: A
) {
    add_group_hom_kernel_quotient_induced(f, add_group_hom_kernel_quotient_project(f, a)) = f.hom(a)
} by {
    kernel_relation_respects(f.hom)
    add_group_hom_kernel_quotient_project_eq_mk(f, a)
    quotient_over_lift_to_projection(kernel_quotient_relation(f.hom), f.hom, a)
}

/// The induced map out of the additive-group-kernel quotient separates projections of unrelated representatives.
theorem add_group_hom_kernel_quotient_induced_injective[A: AddGroup, B: AddGroup](
    f: AddGroupHom[A, B], a: A, b: A
) {
    add_group_hom_kernel_quotient_induced(f, add_group_hom_kernel_quotient_project(f, a)) =
        add_group_hom_kernel_quotient_induced(f, add_group_hom_kernel_quotient_project(f, b))
    implies add_group_hom_kernel_quotient_project(f, a) = add_group_hom_kernel_quotient_project(f, b)
} by {
    if add_group_hom_kernel_quotient_induced(f, add_group_hom_kernel_quotient_project(f, a)) =
        add_group_hom_kernel_quotient_induced(f, add_group_hom_kernel_quotient_project(f, b)) {
        add_group_hom_kernel_quotient_induced_project(f, a)
        add_group_hom_kernel_quotient_induced_project(f, b)
        f.hom(a) = f.hom(b)
        kernel_relation(f.hom, a, b) = (f.hom(a) = f.hom(b))
        kernel_relation(f.hom, a, b)
        add_group_hom_kernel_quotient_project_eq_iff_kernel_relation(f, a, b)
        (add_group_hom_kernel_quotient_project(f, a) = add_group_hom_kernel_quotient_project(f, b)) =
            kernel_relation(f.hom, a, b)
        add_group_hom_kernel_quotient_project(f, a) = add_group_hom_kernel_quotient_project(f, b)
    }
}

/// The canonical additive-group-kernel projection preserves zero.
theorem add_group_hom_kernel_quotient_project_zero[A: AddGroup, B: AddGroup](f: AddGroupHom[A, B]) {
    add_group_hom_kernel_quotient_project(f, A.0) = add_group_hom_kernel_quotient_zero(f)
} by {
    add_group_hom_kernel_quotient_zero_projection(f)
}

/// The canonical additive-group-kernel projection preserves addition.
theorem add_group_hom_kernel_quotient_project_add[A: AddGroup, B: AddGroup](f: AddGroupHom[A, B], a: A, b: A) {
    add_group_hom_kernel_quotient_project(f, a + b) =
    add_group_hom_kernel_quotient_add(f,
        add_group_hom_kernel_quotient_project(f, a),
        add_group_hom_kernel_quotient_project(f, b))
} by {
    add_group_hom_kernel_quotient_add_projection(f, a, b)
    add_group_hom_kernel_quotient_project(f, a + b) =
        quotient_over_mk(kernel_quotient_relation(f.hom), a + b)
    add_group_hom_kernel_quotient_project(f, a) =
        quotient_over_mk(kernel_quotient_relation(f.hom), a)
    add_group_hom_kernel_quotient_project(f, b) =
        quotient_over_mk(kernel_quotient_relation(f.hom), b)
}

/// The canonical additive-group-kernel projection preserves negation.
theorem add_group_hom_kernel_quotient_project_neg[A: AddGroup, B: AddGroup](f: AddGroupHom[A, B], a: A) {
    add_group_hom_kernel_quotient_project(f, -a) =
    add_group_hom_kernel_quotient_neg(f, add_group_hom_kernel_quotient_project(f, a))
} by {
    add_group_hom_kernel_quotient_neg_projection(f, a)
    add_group_hom_kernel_quotient_project(f, -a) =
        quotient_over_mk(kernel_quotient_relation(f.hom), -a)
    add_group_hom_kernel_quotient_project(f, a) =
        quotient_over_mk(kernel_quotient_relation(f.hom), a)
}

/// The canonical additive-group-kernel projection preserves subtraction.
theorem add_group_hom_kernel_quotient_project_sub[A: AddGroup, B: AddGroup](f: AddGroupHom[A, B], a: A, b: A) {
    add_group_hom_kernel_quotient_project(f, a - b) =
    add_group_hom_kernel_quotient_sub(f,
        add_group_hom_kernel_quotient_project(f, a),
        add_group_hom_kernel_quotient_project(f, b))
} by {
    add_group_hom_kernel_quotient_sub_projection(f, a, b)
    add_group_hom_kernel_quotient_project(f, a - b) =
        quotient_over_mk(kernel_quotient_relation(f.hom), a - b)
    add_group_hom_kernel_quotient_project(f, a) =
        quotient_over_mk(kernel_quotient_relation(f.hom), a)
    add_group_hom_kernel_quotient_project(f, b) =
        quotient_over_mk(kernel_quotient_relation(f.hom), b)
}

/// Addition of canonical additive-group-kernel projections respects related left representatives.
theorem add_group_hom_kernel_quotient_project_add_left_of_kernel_relation[A: AddGroup, B: AddGroup](
    f: AddGroupHom[A, B], a1: A, a2: A, b: A
) {
    kernel_relation(f.hom, a1, a2) implies
    add_group_hom_kernel_quotient_add(f,
        add_group_hom_kernel_quotient_project(f, a1),
        add_group_hom_kernel_quotient_project(f, b)) =
    add_group_hom_kernel_quotient_add(f,
        add_group_hom_kernel_quotient_project(f, a2),
        add_group_hom_kernel_quotient_project(f, b))
} by {
    if kernel_relation(f.hom, a1, a2) {
        add_group_hom_kernel_quotient_add_projection_left(f, a1, a2, b)
    }
}

/// Addition of canonical additive-group-kernel projections respects related right representatives.
theorem add_group_hom_kernel_quotient_project_add_right_of_kernel_relation[A: AddGroup, B: AddGroup](
    f: AddGroupHom[A, B], a: A, b1: A, b2: A
) {
    kernel_relation(f.hom, b1, b2) implies
    add_group_hom_kernel_quotient_add(f,
        add_group_hom_kernel_quotient_project(f, a),
        add_group_hom_kernel_quotient_project(f, b1)) =
    add_group_hom_kernel_quotient_add(f,
        add_group_hom_kernel_quotient_project(f, a),
        add_group_hom_kernel_quotient_project(f, b2))
} by {
    if kernel_relation(f.hom, b1, b2) {
        add_group_hom_kernel_quotient_add_projection_right(f, a, b1, b2)
    }
}

/// Negation of canonical additive-group-kernel projections respects related representatives.
theorem add_group_hom_kernel_quotient_project_neg_of_kernel_relation[A: AddGroup, B: AddGroup](
    f: AddGroupHom[A, B], a: A, b: A
) {
    kernel_relation(f.hom, a, b) implies
    add_group_hom_kernel_quotient_neg(f, add_group_hom_kernel_quotient_project(f, a)) =
    add_group_hom_kernel_quotient_neg(f, add_group_hom_kernel_quotient_project(f, b))
} by {
    if kernel_relation(f.hom, a, b) {
        add_group_hom_kernel_quotient_neg_projection_compatible(f, a, b)
    }
}

/// Subtraction of canonical additive-group-kernel projections respects related left representatives.
theorem add_group_hom_kernel_quotient_project_sub_left_of_kernel_relation[A: AddGroup, B: AddGroup](
    f: AddGroupHom[A, B], a1: A, a2: A, b: A
) {
    kernel_relation(f.hom, a1, a2) implies
    add_group_hom_kernel_quotient_sub(f,
        add_group_hom_kernel_quotient_project(f, a1),
        add_group_hom_kernel_quotient_project(f, b)) =
    add_group_hom_kernel_quotient_sub(f,
        add_group_hom_kernel_quotient_project(f, a2),
        add_group_hom_kernel_quotient_project(f, b))
} by {
    if kernel_relation(f.hom, a1, a2) {
        add_group_hom_kernel_quotient_sub_projection_left(f, a1, a2, b)
    }
}

/// Subtraction of canonical additive-group-kernel projections respects related right representatives.
theorem add_group_hom_kernel_quotient_project_sub_right_of_kernel_relation[A: AddGroup, B: AddGroup](
    f: AddGroupHom[A, B], a: A, b1: A, b2: A
) {
    kernel_relation(f.hom, b1, b2) implies
    add_group_hom_kernel_quotient_sub(f,
        add_group_hom_kernel_quotient_project(f, a),
        add_group_hom_kernel_quotient_project(f, b1)) =
    add_group_hom_kernel_quotient_sub(f,
        add_group_hom_kernel_quotient_project(f, a),
        add_group_hom_kernel_quotient_project(f, b2))
} by {
    if kernel_relation(f.hom, b1, b2) {
        add_group_hom_kernel_quotient_sub_projection_right(f, a, b1, b2)
    }
}

/// Addition of projected additive-group-kernel representatives is associative.
theorem add_group_hom_kernel_quotient_project_add_assoc[A: AddGroup, B: AddGroup](
    f: AddGroupHom[A, B], a: A, b: A, c: A
) {
    add_group_hom_kernel_quotient_add(f,
        add_group_hom_kernel_quotient_add(f,
            add_group_hom_kernel_quotient_project(f, a),
            add_group_hom_kernel_quotient_project(f, b)),
        add_group_hom_kernel_quotient_project(f, c)) =
    add_group_hom_kernel_quotient_add(f,
        add_group_hom_kernel_quotient_project(f, a),
        add_group_hom_kernel_quotient_add(f,
            add_group_hom_kernel_quotient_project(f, b),
            add_group_hom_kernel_quotient_project(f, c)))
} by {
    add_group_hom_kernel_quotient_add_assoc(f, a, b, c)
}

/// The quotient zero is a left identity for projected additive-group-kernel representatives.
theorem add_group_hom_kernel_quotient_project_zero_add[A: AddGroup, B: AddGroup](f: AddGroupHom[A, B], a: A) {
    add_group_hom_kernel_quotient_add(f,
        add_group_hom_kernel_quotient_zero(f),
        add_group_hom_kernel_quotient_project(f, a)) =
        add_group_hom_kernel_quotient_project(f, a)
} by {
    add_group_hom_kernel_quotient_add_zero_left_projection(f, a)
}

/// The quotient zero is a right identity for projected additive-group-kernel representatives.
theorem add_group_hom_kernel_quotient_project_add_zero[A: AddGroup, B: AddGroup](f: AddGroupHom[A, B], a: A) {
    add_group_hom_kernel_quotient_add(f,
        add_group_hom_kernel_quotient_project(f, a),
        add_group_hom_kernel_quotient_zero(f)) =
        add_group_hom_kernel_quotient_project(f, a)
} by {
    add_group_hom_kernel_quotient_add_zero_right_projection(f, a)
}

/// The quotient negation is a left inverse for projected additive-group-kernel representatives.
theorem add_group_hom_kernel_quotient_project_neg_add[A: AddGroup, B: AddGroup](f: AddGroupHom[A, B], a: A) {
    add_group_hom_kernel_quotient_add(f,
        add_group_hom_kernel_quotient_neg(f, add_group_hom_kernel_quotient_project(f, a)),
        add_group_hom_kernel_quotient_project(f, a)) =
        add_group_hom_kernel_quotient_zero(f)
} by {
    add_group_hom_kernel_quotient_neg_add(f, a)
    add_group_hom_kernel_quotient_zero_projection(f)
}

/// The quotient negation is a right inverse for projected additive-group-kernel representatives.
theorem add_group_hom_kernel_quotient_project_add_neg[A: AddGroup, B: AddGroup](f: AddGroupHom[A, B], a: A) {
    add_group_hom_kernel_quotient_add(f,
        add_group_hom_kernel_quotient_project(f, a),
        add_group_hom_kernel_quotient_neg(f, add_group_hom_kernel_quotient_project(f, a))) =
        add_group_hom_kernel_quotient_zero(f)
} by {
    add_group_hom_kernel_quotient_add_neg(f, a)
    add_group_hom_kernel_quotient_zero_projection(f)
}

/// Addition of projected additive-group-kernel representatives is commutative in the commutative case.
theorem add_group_hom_kernel_quotient_project_add_comm[A: AddCommGroup, B: AddCommGroup](
    f: AddGroupHom[A, B], a: A, b: A
) {
    add_group_hom_kernel_quotient_add(f,
        add_group_hom_kernel_quotient_project(f, a),
        add_group_hom_kernel_quotient_project(f, b)) =
    add_group_hom_kernel_quotient_add(f,
        add_group_hom_kernel_quotient_project(f, b),
        add_group_hom_kernel_quotient_project(f, a))
} by {
    add_group_hom_kernel_quotient_add_comm(f, a, b)
}

/// Subtracting quotient zero fixes a projected additive-group-kernel representative.
theorem add_group_hom_kernel_quotient_project_sub_zero[A: AddGroup, B: AddGroup](
    f: AddGroupHom[A, B], a: A
) {
    add_group_hom_kernel_quotient_sub(f,
        add_group_hom_kernel_quotient_project(f, a),
        add_group_hom_kernel_quotient_zero(f)) =
        add_group_hom_kernel_quotient_project(f, a)
} by {
    add_group_hom_kernel_quotient_sub_zero(f, a)
    add_group_hom_kernel_quotient_zero_projection(f)
}

/// Subtracting a projected representative from itself gives quotient zero.
theorem add_group_hom_kernel_quotient_project_sub_self[A: AddGroup, B: AddGroup](
    f: AddGroupHom[A, B], a: A
) {
    add_group_hom_kernel_quotient_sub(f,
        add_group_hom_kernel_quotient_project(f, a),
        add_group_hom_kernel_quotient_project(f, a)) =
        add_group_hom_kernel_quotient_zero(f)
} by {
    add_group_hom_kernel_quotient_sub_self(f, a)
    add_group_hom_kernel_quotient_zero_projection(f)
}

/// Subtracting a projected representative from zero is quotient negation.
theorem add_group_hom_kernel_quotient_project_zero_sub[A: AddGroup, B: AddGroup](
    f: AddGroupHom[A, B], a: A
) {
    add_group_hom_kernel_quotient_sub(f,
        add_group_hom_kernel_quotient_zero(f),
        add_group_hom_kernel_quotient_project(f, a)) =
        add_group_hom_kernel_quotient_neg(f,
            add_group_hom_kernel_quotient_project(f, a))
} by {
    add_group_hom_kernel_quotient_zero_sub(f, a)
    add_group_hom_kernel_quotient_zero_projection(f)
}

/// Projected quotient subtraction is addition of quotient negation.
theorem add_group_hom_kernel_quotient_project_sub_eq_add_neg[A: AddGroup, B: AddGroup](
    f: AddGroupHom[A, B], a: A, b: A
) {
    add_group_hom_kernel_quotient_sub(f,
        add_group_hom_kernel_quotient_project(f, a),
        add_group_hom_kernel_quotient_project(f, b)) =
    add_group_hom_kernel_quotient_add(f,
        add_group_hom_kernel_quotient_project(f, a),
        add_group_hom_kernel_quotient_neg(f,
            add_group_hom_kernel_quotient_project(f, b)))
} by {
    add_group_hom_kernel_quotient_sub_eq_add_neg(f, a, b)
}

/// A projected quotient negation cancels a matching left summand.
theorem add_group_hom_kernel_quotient_project_neg_add_cancel_left[A: AddGroup, B: AddGroup](
    f: AddGroupHom[A, B], a: A, b: A
) {
    add_group_hom_kernel_quotient_add(f,
        add_group_hom_kernel_quotient_neg(f,
            add_group_hom_kernel_quotient_project(f, a)),
        add_group_hom_kernel_quotient_add(f,
            add_group_hom_kernel_quotient_project(f, a),
            add_group_hom_kernel_quotient_project(f, b))) =
        add_group_hom_kernel_quotient_project(f, b)
} by {
    add_group_hom_kernel_quotient_neg_add_cancel_left(f, a, b)
}

/// A projected quotient negation cancels a matching right summand.
theorem add_group_hom_kernel_quotient_project_add_neg_cancel_right[A: AddGroup, B: AddGroup](
    f: AddGroupHom[A, B], a: A, b: A
) {
    add_group_hom_kernel_quotient_add(f,
        add_group_hom_kernel_quotient_add(f,
            add_group_hom_kernel_quotient_project(f, b),
            add_group_hom_kernel_quotient_project(f, a)),
        add_group_hom_kernel_quotient_neg(f,
            add_group_hom_kernel_quotient_project(f, a))) =
        add_group_hom_kernel_quotient_project(f, b)
} by {
    add_group_hom_kernel_quotient_add_neg_cancel_right(f, a, b)
}

/// Subtracting the right summand of a projected quotient sum cancels it.
theorem add_group_hom_kernel_quotient_project_add_sub_cancel_right[A: AddGroup, B: AddGroup](
    f: AddGroupHom[A, B], a: A, b: A
) {
    add_group_hom_kernel_quotient_sub(f,
        add_group_hom_kernel_quotient_add(f,
            add_group_hom_kernel_quotient_project(f, a),
            add_group_hom_kernel_quotient_project(f, b)),
        add_group_hom_kernel_quotient_project(f, b)) =
        add_group_hom_kernel_quotient_project(f, a)
} by {
    add_group_hom_kernel_quotient_add_sub_cancel_right(f, a, b)
}

/// Subtracting the left summand of a projected additive-group-kernel quotient sum cancels it.
theorem add_group_hom_kernel_quotient_project_add_sub_cancel_left[A: AddGroup, B: AddGroup](
    f: AddGroupHom[A, B], a: A, b: A
) {
    add_group_hom_kernel_quotient_sub(f,
        add_group_hom_kernel_quotient_add(f,
            add_group_hom_kernel_quotient_project(f, a),
            add_group_hom_kernel_quotient_project(f, b)),
        add_group_hom_kernel_quotient_project(f, a)) =
        add_group_hom_kernel_quotient_project(f, b)
} by {
    add_group_hom_kernel_quotient_add_sub_cancel_left(f, a, b)
}

/// Additive-group-kernel quotient negation sends quotient zero to quotient zero.
theorem add_group_hom_kernel_quotient_project_neg_zero[A: AddGroup, B: AddGroup](f: AddGroupHom[A, B]) {
    add_group_hom_kernel_quotient_neg(f, add_group_hom_kernel_quotient_zero(f)) =
        add_group_hom_kernel_quotient_zero(f)
} by {
    add_group_hom_kernel_quotient_neg_zero(f)
}

/// Additive-group-kernel quotient negation is involutive on projected representatives.
theorem add_group_hom_kernel_quotient_project_neg_neg[A: AddGroup, B: AddGroup](
    f: AddGroupHom[A, B], a: A
) {
    add_group_hom_kernel_quotient_neg(f,
        add_group_hom_kernel_quotient_neg(f, add_group_hom_kernel_quotient_project(f, a))) =
        add_group_hom_kernel_quotient_project(f, a)
} by {
    add_group_hom_kernel_quotient_neg_neg(f, a)
}

/// The induced map out of the additive-group-kernel quotient preserves addition.
theorem add_group_hom_kernel_quotient_induced_add[A: AddGroup, B: AddGroup](
    f: AddGroupHom[A, B], a: A, b: A
) {
    add_group_hom_kernel_quotient_induced(f,
        add_group_hom_kernel_quotient_add(f,
            add_group_hom_kernel_quotient_project(f, a),
            add_group_hom_kernel_quotient_project(f, b))) =
        add_group_hom_kernel_quotient_induced(f, add_group_hom_kernel_quotient_project(f, a)) +
        add_group_hom_kernel_quotient_induced(f, add_group_hom_kernel_quotient_project(f, b))
} by {
    add_group_hom_kernel_quotient_project_add(f, a, b)
    add_group_hom_kernel_quotient_induced_project(f, a + b)
    add_group_hom_kernel_quotient_induced_project(f, a)
    add_group_hom_kernel_quotient_induced_project(f, b)
    add_group_hom_add(f, a, b)
}

/// The induced map out of the additive-group-kernel quotient preserves zero.
theorem add_group_hom_kernel_quotient_induced_zero[A: AddGroup, B: AddGroup](f: AddGroupHom[A, B]) {
    add_group_hom_kernel_quotient_induced(f, add_group_hom_kernel_quotient_zero(f)) = B.0
} by {
    add_group_hom_kernel_quotient_project_zero(f)
    add_group_hom_kernel_quotient_induced_project(f, A.0)
    add_group_hom_zero(f)
}

/// The induced map out of the additive-group-kernel quotient preserves negation.
theorem add_group_hom_kernel_quotient_induced_neg[A: AddGroup, B: AddGroup](f: AddGroupHom[A, B], a: A) {
    add_group_hom_kernel_quotient_induced(f,
        add_group_hom_kernel_quotient_neg(f, add_group_hom_kernel_quotient_project(f, a))) =
        -add_group_hom_kernel_quotient_induced(f, add_group_hom_kernel_quotient_project(f, a))
} by {
    add_group_hom_kernel_quotient_project_neg(f, a)
    add_group_hom_kernel_quotient_induced_project(f, -a)
    add_group_hom_kernel_quotient_induced_project(f, a)
    add_group_hom_neg(f, a)
}

/// Every value of `f` is attained by the induced map out of the additive-group-kernel quotient.
theorem add_group_hom_kernel_quotient_induced_surjective_onto_image[A: AddGroup, B: AddGroup](
    f: AddGroupHom[A, B], a: A
) {
    exists(q: QuotientOver[A]) {
        add_group_hom_kernel_quotient_induced(f, q) = f.hom(a)
    }
} by {
    add_group_hom_kernel_quotient_induced_project(f, a)
    add_group_hom_kernel_quotient_induced(f, add_group_hom_kernel_quotient_project(f, a)) = f.hom(a)
}

/// Multiplication of canonical group-kernel projections respects related left representatives.
theorem group_hom_kernel_quotient_project_mul_left_of_kernel_relation[G: Group, H: Group](
    f: GroupHom[G, H], a1: G, a2: G, b: G
) {
    kernel_relation(f.hom, a1, a2) implies
    group_hom_kernel_quotient_mul(f,
        group_hom_kernel_quotient_project(f, a1),
        group_hom_kernel_quotient_project(f, b)) =
    group_hom_kernel_quotient_mul(f,
        group_hom_kernel_quotient_project(f, a2),
        group_hom_kernel_quotient_project(f, b))
} by {
    if kernel_relation(f.hom, a1, a2) {
        group_hom_kernel_quotient_mul_projection_left(f, a1, a2, b)
    }
}

/// Multiplication of canonical group-kernel projections respects related right representatives.
theorem group_hom_kernel_quotient_project_mul_right_of_kernel_relation[G: Group, H: Group](
    f: GroupHom[G, H], a: G, b1: G, b2: G
) {
    kernel_relation(f.hom, b1, b2) implies
    group_hom_kernel_quotient_mul(f,
        group_hom_kernel_quotient_project(f, a),
        group_hom_kernel_quotient_project(f, b1)) =
    group_hom_kernel_quotient_mul(f,
        group_hom_kernel_quotient_project(f, a),
        group_hom_kernel_quotient_project(f, b2))
} by {
    if kernel_relation(f.hom, b1, b2) {
        group_hom_kernel_quotient_mul_projection_right(f, a, b1, b2)
    }
}

/// Inversion of canonical group-kernel projections respects related representatives.
theorem group_hom_kernel_quotient_project_inverse_of_kernel_relation[G: Group, H: Group](
    f: GroupHom[G, H], a: G, b: G
) {
    kernel_relation(f.hom, a, b) implies
    group_hom_kernel_quotient_inverse(f, group_hom_kernel_quotient_project(f, a)) =
        group_hom_kernel_quotient_inverse(f, group_hom_kernel_quotient_project(f, b))
} by {
    if kernel_relation(f.hom, a, b) {
        group_hom_kernel_quotient_inverse_projection_compatible(f, a, b)
    }
}

/// Addition of canonical ring-kernel projections respects related left representatives.
theorem ring_hom_kernel_quotient_project_add_left_of_kernel_relation[R: Ring, S: Ring](
    f: RingHom[R, S], a1: R, a2: R, b: R
) {
    kernel_relation(f.hom, a1, a2) implies
    ring_hom_kernel_quotient_add(f,
        ring_hom_kernel_quotient_project(f, a1),
        ring_hom_kernel_quotient_project(f, b)) =
    ring_hom_kernel_quotient_add(f,
        ring_hom_kernel_quotient_project(f, a2),
        ring_hom_kernel_quotient_project(f, b))
} by {
    if kernel_relation(f.hom, a1, a2) {
        ring_hom_kernel_quotient_add_projection_left(f, a1, a2, b)
    }
}

/// Addition of canonical ring-kernel projections respects related right representatives.
theorem ring_hom_kernel_quotient_project_add_right_of_kernel_relation[R: Ring, S: Ring](
    f: RingHom[R, S], a: R, b1: R, b2: R
) {
    kernel_relation(f.hom, b1, b2) implies
    ring_hom_kernel_quotient_add(f,
        ring_hom_kernel_quotient_project(f, a),
        ring_hom_kernel_quotient_project(f, b1)) =
    ring_hom_kernel_quotient_add(f,
        ring_hom_kernel_quotient_project(f, a),
        ring_hom_kernel_quotient_project(f, b2))
} by {
    if kernel_relation(f.hom, b1, b2) {
        ring_hom_kernel_quotient_add_projection_right(f, a, b1, b2)
    }
}

/// Multiplication of canonical ring-kernel projections respects related left representatives.
theorem ring_hom_kernel_quotient_project_mul_left_of_kernel_relation[R: Ring, S: Ring](
    f: RingHom[R, S], a1: R, a2: R, b: R
) {
    kernel_relation(f.hom, a1, a2) implies
    ring_hom_kernel_quotient_mul(f,
        ring_hom_kernel_quotient_project(f, a1),
        ring_hom_kernel_quotient_project(f, b)) =
    ring_hom_kernel_quotient_mul(f,
        ring_hom_kernel_quotient_project(f, a2),
        ring_hom_kernel_quotient_project(f, b))
} by {
    if kernel_relation(f.hom, a1, a2) {
        ring_hom_kernel_quotient_mul_projection_left(f, a1, a2, b)
    }
}

/// Multiplication of canonical ring-kernel projections respects related right representatives.
theorem ring_hom_kernel_quotient_project_mul_right_of_kernel_relation[R: Ring, S: Ring](
    f: RingHom[R, S], a: R, b1: R, b2: R
) {
    kernel_relation(f.hom, b1, b2) implies
    ring_hom_kernel_quotient_mul(f,
        ring_hom_kernel_quotient_project(f, a),
        ring_hom_kernel_quotient_project(f, b1)) =
    ring_hom_kernel_quotient_mul(f,
        ring_hom_kernel_quotient_project(f, a),
        ring_hom_kernel_quotient_project(f, b2))
} by {
    if kernel_relation(f.hom, b1, b2) {
        ring_hom_kernel_quotient_mul_projection_right(f, a, b1, b2)
    }
}

/// Negation of canonical ring-kernel projections respects related representatives.
theorem ring_hom_kernel_quotient_project_neg_of_kernel_relation[R: Ring, S: Ring](
    f: RingHom[R, S], a: R, b: R
) {
    kernel_relation(f.hom, a, b) implies
    ring_hom_kernel_quotient_neg(f, ring_hom_kernel_quotient_project(f, a)) =
        ring_hom_kernel_quotient_neg(f, ring_hom_kernel_quotient_project(f, b))
} by {
    if kernel_relation(f.hom, a, b) {
        ring_hom_kernel_quotient_neg_projection_compatible(f, a, b)
    }
}

/// Subtraction of canonical ring-kernel projections respects related left representatives.
theorem ring_hom_kernel_quotient_project_sub_left_of_kernel_relation[R: Ring, S: Ring](
    f: RingHom[R, S], a1: R, a2: R, b: R
) {
    kernel_relation(f.hom, a1, a2) implies
    ring_hom_kernel_quotient_sub(f,
        ring_hom_kernel_quotient_project(f, a1),
        ring_hom_kernel_quotient_project(f, b)) =
    ring_hom_kernel_quotient_sub(f,
        ring_hom_kernel_quotient_project(f, a2),
        ring_hom_kernel_quotient_project(f, b))
} by {
    if kernel_relation(f.hom, a1, a2) {
        ring_hom_kernel_quotient_sub_projection_left(f, a1, a2, b)
    }
}

/// Subtraction of canonical ring-kernel projections respects related right representatives.
theorem ring_hom_kernel_quotient_project_sub_right_of_kernel_relation[R: Ring, S: Ring](
    f: RingHom[R, S], a: R, b1: R, b2: R
) {
    kernel_relation(f.hom, b1, b2) implies
    ring_hom_kernel_quotient_sub(f,
        ring_hom_kernel_quotient_project(f, a),
        ring_hom_kernel_quotient_project(f, b1)) =
    ring_hom_kernel_quotient_sub(f,
        ring_hom_kernel_quotient_project(f, a),
        ring_hom_kernel_quotient_project(f, b2))
} by {
    if kernel_relation(f.hom, b1, b2) {
        ring_hom_kernel_quotient_sub_projection_right(f, a, b1, b2)
    }
}

/// The induced map out of the semiring-kernel quotient identifies projected
/// representatives exactly when their homomorphism values agree.
theorem semiring_hom_kernel_quotient_induced_eq_iff_image_eq[S: Semiring, T: Semiring](
    f: SemiringHom[S, T], a: S, b: S
) {
    (semiring_hom_kernel_quotient_induced(f, semiring_hom_kernel_quotient_project(f, a)) =
        semiring_hom_kernel_quotient_induced(f, semiring_hom_kernel_quotient_project(f, b))) =
        (f.hom(a) = f.hom(b))
} by {
    semiring_hom_kernel_quotient_induced_project(f, a)
    semiring_hom_kernel_quotient_induced_project(f, b)
}

/// The induced map out of the ring-kernel quotient identifies projected
/// representatives exactly when their homomorphism values agree.
theorem ring_hom_kernel_quotient_induced_eq_iff_image_eq[R: Ring, S: Ring](
    f: RingHom[R, S], a: R, b: R
) {
    (ring_hom_kernel_quotient_induced(f, ring_hom_kernel_quotient_project(f, a)) =
        ring_hom_kernel_quotient_induced(f, ring_hom_kernel_quotient_project(f, b))) =
        (f.hom(a) = f.hom(b))
} by {
    ring_hom_kernel_quotient_induced_project(f, a)
    ring_hom_kernel_quotient_induced_project(f, b)
}

/// The induced map out of the group-kernel quotient identifies projected
/// representatives exactly when their homomorphism values agree.
theorem group_hom_kernel_quotient_induced_eq_iff_image_eq[G: Group, H: Group](
    f: GroupHom[G, H], a: G, b: G
) {
    (group_hom_kernel_quotient_induced(f, group_hom_kernel_quotient_project(f, a)) =
        group_hom_kernel_quotient_induced(f, group_hom_kernel_quotient_project(f, b))) =
        (f.hom(a) = f.hom(b))
} by {
    group_hom_kernel_quotient_induced_project(f, a)
    group_hom_kernel_quotient_induced_project(f, b)
}

/// The induced map out of the monoid-kernel quotient identifies projected
/// representatives exactly when their homomorphism values agree.
theorem monoid_hom_kernel_quotient_induced_eq_iff_image_eq[M: Monoid, N: Monoid](
    f: MonoidHom[M, N], a: M, b: M
) {
    (monoid_hom_kernel_quotient_induced(f, monoid_hom_kernel_quotient_project(f, a)) =
        monoid_hom_kernel_quotient_induced(f, monoid_hom_kernel_quotient_project(f, b))) =
        (f.hom(a) = f.hom(b))
} by {
    monoid_hom_kernel_quotient_induced_project(f, a)
    monoid_hom_kernel_quotient_induced_project(f, b)
}

/// The induced map out of the additive-monoid-kernel quotient identifies projected
/// representatives exactly when their homomorphism values agree.
theorem add_monoid_hom_kernel_quotient_induced_eq_iff_image_eq[A: AddMonoid, B: AddMonoid](
    f: AddMonoidHom[A, B], a: A, b: A
) {
    (add_monoid_hom_kernel_quotient_induced(f, add_monoid_hom_kernel_quotient_project(f, a)) =
        add_monoid_hom_kernel_quotient_induced(f, add_monoid_hom_kernel_quotient_project(f, b))) =
        (f.hom(a) = f.hom(b))
} by {
    add_monoid_hom_kernel_quotient_induced_project(f, a)
    add_monoid_hom_kernel_quotient_induced_project(f, b)
}

/// The induced map out of the additive-group-kernel quotient identifies projected
/// representatives exactly when their homomorphism values agree.
theorem add_group_hom_kernel_quotient_induced_eq_iff_image_eq[A: AddGroup, B: AddGroup](
    f: AddGroupHom[A, B], a: A, b: A
) {
    (add_group_hom_kernel_quotient_induced(f, add_group_hom_kernel_quotient_project(f, a)) =
        add_group_hom_kernel_quotient_induced(f, add_group_hom_kernel_quotient_project(f, b))) =
        (f.hom(a) = f.hom(b))
} by {
    add_group_hom_kernel_quotient_induced_project(f, a)
    add_group_hom_kernel_quotient_induced_project(f, b)
}

/// The induced map out of the linear-map-kernel quotient identifies projected
/// representatives exactly when their values agree.
theorem linear_map_kernel_quotient_induced_eq_iff_image_eq[R: Ring, M: AddCommGroup, N: AddCommGroup](
    src: Module[R, M], dst: Module[R, N], f: M -> N, a: M, b: M
) {
    (linear_map_kernel_quotient_induced(src, dst, f,
        linear_map_kernel_quotient_project(src, dst, f, a)) =
        linear_map_kernel_quotient_induced(src, dst, f,
            linear_map_kernel_quotient_project(src, dst, f, b))) =
        (f(a) = f(b))
} by {
    linear_map_kernel_quotient_induced_project(src, dst, f, a)
    linear_map_kernel_quotient_induced_project(src, dst, f, b)
}
