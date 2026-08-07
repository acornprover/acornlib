from data.basic.set import Set, empty_set_contains_eq, universal_set_contains_eq, compl_contains_eq,
    difference_contains_eq, difference_contains_intro, difference_subset, empty_set_is_always_subset,
    intersection_contains_eq, subset_contains, subset_antisymm, set_ext

/// Removing the empty set changes nothing.
theorem difference_empty_right[K](s: Set[K]) {
    s.difference(Set[K].empty_set) = s
} by {
    difference_subset(s, Set[K].empty_set)
    s.difference(Set[K].empty_set).subset(s)
    forall(x: K) {
        if s.contains(x) {
            empty_set_contains_eq[K](x)
            not Set[K].empty_set.contains(x)
            difference_contains_intro(s, Set[K].empty_set, x)
            s.difference(Set[K].empty_set).contains(x)
        }
    }
    s.subset(s.difference(Set[K].empty_set))
    subset_antisymm(s.difference(Set[K].empty_set), s)
}

/// The empty set minus any set is empty.
theorem empty_difference_left[K](s: Set[K]) {
    Set[K].empty_set.difference(s) = Set[K].empty_set
} by {
    difference_subset(Set[K].empty_set, s)
    Set[K].empty_set.difference(s).subset(Set[K].empty_set)
    empty_set_is_always_subset[K](Set[K].empty_set.difference(s))
    Set[K].empty_set.subset(Set[K].empty_set.difference(s))
    subset_antisymm(Set[K].empty_set.difference(s), Set[K].empty_set)
}

/// Removing the universal set leaves the empty set.
theorem difference_universal_right[K](s: Set[K]) {
    s.difference(Set[K].universal_set) = Set[K].empty_set
} by {
    forall(x: K) {
        if s.difference(Set[K].universal_set).contains(x) {
            difference_contains_eq(s, Set[K].universal_set, x)
            not Set[K].universal_set.contains(x)
            universal_set_contains_eq[K](x)
            false
        }
    }
    s.difference(Set[K].universal_set).subset(Set[K].empty_set)
    empty_set_is_always_subset[K](s.difference(Set[K].universal_set))
    Set[K].empty_set.subset(s.difference(Set[K].universal_set))
    subset_antisymm(s.difference(Set[K].universal_set), Set[K].empty_set)
}

/// The universal set minus a set is its complement.
theorem universal_difference_eq_complement[K](s: Set[K]) {
    Set[K].universal_set.difference(s) = s.c
} by {
    forall(x: K) {
        difference_contains_eq(Set[K].universal_set, s, x)
        universal_set_contains_eq[K](x)
        compl_contains_eq(s, x)
        if s.contains(x) {
            not s.c.contains(x)
            not Set[K].universal_set.difference(s).contains(x)
            Set[K].universal_set.difference(s).contains(x) = s.c.contains(x)
        } else {
            s.c.contains(x)
            Set[K].universal_set.difference(s).contains(x)
            Set[K].universal_set.difference(s).contains(x) = s.c.contains(x)
        }
    }
    set_ext(Set[K].universal_set.difference(s), s.c)
}

/// Removing a complement is intersecting with the original set.
theorem difference_complement_right_eq_intersection[K](s: Set[K], t: Set[K]) {
    s.difference(t.c) = s.intersection(t)
} by {
    forall(x: K) {
        difference_contains_eq(s, t.c, x)
        compl_contains_eq(t, x)
        intersection_contains_eq(s, t, x)
        if t.contains(x) {
            not t.c.contains(x)
            if s.contains(x) {
                s.difference(t.c).contains(x)
                s.intersection(t).contains(x)
                s.difference(t.c).contains(x) = s.intersection(t).contains(x)
            } else {
                not s.difference(t.c).contains(x)
                not s.intersection(t).contains(x)
                s.difference(t.c).contains(x) = s.intersection(t).contains(x)
            }
        } else {
            t.c.contains(x)
            not s.difference(t.c).contains(x)
            not s.intersection(t).contains(x)
            s.difference(t.c).contains(x) = s.intersection(t).contains(x)
        }
    }
    set_ext(s.difference(t.c), s.intersection(t))
}

/// Subtracting a superset leaves the empty set.
theorem difference_of_subset_eq_empty[K](s: Set[K], t: Set[K]) {
    s.subset(t) implies s.difference(t) = Set[K].empty_set
} by {
    if s.subset(t) {
        forall(x: K) {
            if s.difference(t).contains(x) {
                difference_contains_eq(s, t, x)
                s.contains(x)
                not t.contains(x)
                subset_contains(s, t, x)
                t.contains(x)
                false
            }
            not s.difference(t).contains(x)
            empty_set_contains_eq[K](x)
            s.difference(t).contains(x) = Set[K].empty_set.contains(x)
        }
        set_ext(s.difference(t), Set[K].empty_set)
        s.difference(t) = Set[K].empty_set
    }
}
