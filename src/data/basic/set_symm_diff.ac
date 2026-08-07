/// Symmetric difference for sets.

from data.basic.set import Set, set_ext, empty_set_contains_eq, set_eq_empty_of_is_empty,
    subset_contains_eq, union_contains_eq, difference_contains_eq

/// Elements that belong to exactly one of two sets.
define symm_diff_contains[K](s: Set[K], t: Set[K], x: K) -> Bool {
    (s.contains(x) and not t.contains(x)) or (t.contains(x) and not s.contains(x))
}

/// The symmetric difference of two sets.
define symm_diff[K](s: Set[K], t: Set[K]) -> Set[K] {
    Set[K].new(symm_diff_contains(s, t))
}

/// Membership in a symmetric difference is membership in exactly one side.
theorem mem_symm_diff[K](s: Set[K], t: Set[K], x: K) {
    symm_diff(s, t).contains(x) iff
        (s.contains(x) and not t.contains(x)) or (t.contains(x) and not s.contains(x))
} by {
    if symm_diff(s, t).contains(x) {
        symm_diff(s, t).contains(x) = symm_diff_contains(s, t, x)
        symm_diff_contains(s, t, x) =
            (s.contains(x) and not t.contains(x)) or (t.contains(x) and not s.contains(x))
        (s.contains(x) and not t.contains(x)) or (t.contains(x) and not s.contains(x))
    }
    if (s.contains(x) and not t.contains(x)) or (t.contains(x) and not s.contains(x)) {
        symm_diff_contains(s, t, x) =
            (s.contains(x) and not t.contains(x)) or (t.contains(x) and not s.contains(x))
        symm_diff(s, t).contains(x) = symm_diff_contains(s, t, x)
        symm_diff(s, t).contains(x)
    }
}

/// Membership in a symmetric difference as Boolean equality.
theorem symm_diff_contains_eq[K](s: Set[K], t: Set[K], x: K) {
    symm_diff(s, t).contains(x) =
        ((s.contains(x) and not t.contains(x)) or (t.contains(x) and not s.contains(x)))
} by {
    if symm_diff(s, t).contains(x) {
        mem_symm_diff(s, t, x)
        (s.contains(x) and not t.contains(x)) or (t.contains(x) and not s.contains(x))
    }
    if (s.contains(x) and not t.contains(x)) or (t.contains(x) and not s.contains(x)) {
        mem_symm_diff(s, t, x)
        symm_diff(s, t).contains(x)
    }
}

/// The symmetric difference is the union of the two directed differences.
theorem symm_diff_def[K](s: Set[K], t: Set[K]) {
    symm_diff(s, t) = s.difference(t).union(t.difference(s))
} by {
    forall(x: K) {
        symm_diff_contains_eq(s, t, x)
        difference_contains_eq(s, t, x)
        difference_contains_eq(t, s, x)
        union_contains_eq(s.difference(t), t.difference(s), x)
        symm_diff(s, t).contains(x) = s.difference(t).union(t.difference(s)).contains(x)
    }
    set_ext(symm_diff(s, t), s.difference(t).union(t.difference(s)))
}

/// Boolean bi-implication for set membership.
define set_bihimp_contains[K](s: Set[K], t: Set[K], x: K) -> Bool {
    s.contains(x) = t.contains(x)
}

/// The set of points where two sets have the same membership value.
define set_bihimp[K](s: Set[K], t: Set[K]) -> Set[K] {
    Set[K].new(set_bihimp_contains(s, t))
}

/// Membership in the Boolean bi-implication is logical equivalence of membership.
theorem mem_bihimp_iff[K](s: Set[K], t: Set[K], x: K) {
    set_bihimp(s, t).contains(x) iff (s.contains(x) iff t.contains(x))
} by {
    if set_bihimp(s, t).contains(x) {
        set_bihimp(s, t).contains(x) = set_bihimp_contains(s, t, x)
        set_bihimp_contains(s, t, x) = (s.contains(x) = t.contains(x))
        s.contains(x) = t.contains(x)
        if s.contains(x) {
            t.contains(x)
        }
        if t.contains(x) {
            s.contains(x)
        }
    }
    if s.contains(x) iff t.contains(x) {
        if s.contains(x) {
            t.contains(x)
            s.contains(x) = t.contains(x)
        } else {
            if t.contains(x) {
                s.contains(x)
                false
            }
            not t.contains(x)
            s.contains(x) = t.contains(x)
        }
        set_bihimp_contains(s, t, x) = (s.contains(x) = t.contains(x))
        set_bihimp(s, t).contains(x) = set_bihimp_contains(s, t, x)
        set_bihimp(s, t).contains(x)
    }
}

/// The symmetric difference is contained in the union.
theorem symm_diff_subset_union[K](s: Set[K], t: Set[K]) {
    symm_diff(s, t).subset(s.union(t))
} by {
    forall(x: K) {
        if symm_diff(s, t).contains(x) {
            symm_diff_contains_eq(s, t, x)
            if s.contains(x) and not t.contains(x) {
                s.contains(x)
            }
            if t.contains(x) and not s.contains(x) {
                t.contains(x)
            }
            s.contains(x) or t.contains(x)
            union_contains_eq(s, t, x)
            s.union(t).contains(x)
        }
    }
}

/// Empty symmetric difference gives equal membership at a point.
theorem symm_diff_eq_empty_contains_eq[K](s: Set[K], t: Set[K], x: K) {
    symm_diff(s, t) = Set[K].empty_set implies s.contains(x) = t.contains(x)
} by {
    if symm_diff(s, t) = Set[K].empty_set {
        if s.contains(x) {
            if not t.contains(x) {
                symm_diff_contains_eq(s, t, x)
                symm_diff(s, t).contains(x)
                empty_set_contains_eq[K](x)
                false
            }
            t.contains(x)
        }
        if t.contains(x) {
            if not s.contains(x) {
                symm_diff_contains_eq(s, t, x)
                symm_diff(s, t).contains(x)
                empty_set_contains_eq[K](x)
                false
            }
            s.contains(x)
        }
        s.contains(x) = t.contains(x)
    }
}

/// If the symmetric difference is empty, the two sets are equal.
theorem eq_of_symm_diff_eq_empty[K](s: Set[K], t: Set[K]) {
    symm_diff(s, t) = Set[K].empty_set implies s = t
} by {
    if symm_diff(s, t) = Set[K].empty_set {
        forall(x: K) {
            symm_diff_eq_empty_contains_eq(s, t, x)
        }
        set_ext(s, t)
    }
}

/// Equal sets exclude membership in their symmetric difference at a point.
theorem not_symm_diff_contains_of_eq[K](s: Set[K], t: Set[K], x: K) {
    s = t implies not symm_diff(s, t).contains(x)
} by {
    if s = t {
        if symm_diff(s, t).contains(x) {
            symm_diff_contains_eq(s, t, x)
            if s.contains(x) and not t.contains(x) {
                false
            }
            if t.contains(x) and not s.contains(x) {
                false
            }
            false
        }
    }
}

/// Equal sets have empty symmetric difference.
theorem symm_diff_eq_empty_of_eq[K](s: Set[K], t: Set[K]) {
    s = t implies symm_diff(s, t) = Set[K].empty_set
} by {
    if s = t {
        forall(x: K) {
            not_symm_diff_contains_of_eq(s, t, x)
            empty_set_contains_eq[K](x)
            symm_diff(s, t).contains(x) = Set[K].empty_set.contains(x)
        }
        set_ext(symm_diff(s, t), Set[K].empty_set)
    }
}

/// A symmetric difference is empty exactly when the sets are equal.
theorem symm_diff_eq_empty[K](s: Set[K], t: Set[K]) {
    symm_diff(s, t) = Set[K].empty_set iff s = t
} by {
    if symm_diff(s, t) = Set[K].empty_set {
        eq_of_symm_diff_eq_empty(s, t)
    }
    if s = t {
        symm_diff_eq_empty_of_eq(s, t)
    }
}

/// A set is nonempty when it is not the empty set.
define set_nonempty[K](s: Set[K]) -> Bool {
    s != Set[K].empty_set
}

/// Nonemptiness unfolds to inequality with the empty set.
theorem set_nonempty_iff_ne_empty[K](s: Set[K]) {
    set_nonempty(s) iff s != Set[K].empty_set
} by {
    if set_nonempty(s) {
        set_nonempty(s) = (s != Set[K].empty_set)
        s != Set[K].empty_set
    }
    if s != Set[K].empty_set {
        set_nonempty(s) = (s != Set[K].empty_set)
        set_nonempty(s)
    }
}

/// The symmetric difference is nonempty exactly when the two sets are unequal.
theorem symm_diff_nonempty[K](s: Set[K], t: Set[K]) {
    set_nonempty(symm_diff(s, t)) iff s != t
} by {
    if set_nonempty(symm_diff(s, t)) {
        set_nonempty_iff_ne_empty(symm_diff(s, t))
        symm_diff(s, t) != Set[K].empty_set
        if s = t {
            symm_diff_eq_empty(s, t)
            symm_diff(s, t) = Set[K].empty_set
            false
        }
        s != t
    }
    if s != t {
        if not set_nonempty(symm_diff(s, t)) {
            set_nonempty_iff_ne_empty(symm_diff(s, t))
            not symm_diff(s, t) != Set[K].empty_set
            symm_diff(s, t) = Set[K].empty_set
            symm_diff_eq_empty(s, t)
            s = t
            false
        }
        set_nonempty(symm_diff(s, t))
    }
}
