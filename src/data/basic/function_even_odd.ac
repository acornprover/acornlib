/// Even and odd functions with respect to negation on the domain.

from algebra.add import Add
from algebra.add_comm_group import AddCommGroup
from algebra.add_group import AddGroup, inverse_add
from data.basic.function_algebra import pointwise_zero, pointwise_add, pointwise_mul
from data.basic.functions import compose
from algebra.mul import Mul
from algebra.neg import Neg
from algebra.zero import Zero

/// True if a function is invariant under negating its argument.
define function_even[A: Neg, B](f: A -> B) -> Bool {
    forall(x: A) {
        f(-x) = f(x)
    }
}

/// True if a function changes sign when its argument is negated.
define function_odd[A: Neg, B: Neg](f: A -> B) -> Bool {
    forall(x: A) {
        f(-x) = -f(x)
    }
}

/// The definition of an even function unfolds to equality at every negated argument.
theorem function_even_def[A: Neg, B](f: A -> B) {
    function_even(f) = forall(x: A) {
        f(-x) = f(x)
    }
} by {
    function_even(f) = forall(x: A) {
        f(-x) = f(x)
    }
}

/// An even function has equal values at an argument and its negation.
theorem function_even_eq[A: Neg, B](f: A -> B, x: A) {
    function_even(f) implies f(-x) = f(x)
} by {
    if function_even(f) {
        function_even_def(f)
        forall(y: A) {
            f(-y) = f(y)
        }
        f(-x) = f(x)
    }
}

/// The definition of an odd function unfolds to negation at every negated argument.
theorem function_odd_def[A: Neg, B: Neg](f: A -> B) {
    function_odd(f) = forall(x: A) {
        f(-x) = -f(x)
    }
} by {
    function_odd(f) = forall(x: A) {
        f(-x) = -f(x)
    }
}

/// An odd function negates values at a negated argument.
theorem function_odd_eq[A: Neg, B: Neg](f: A -> B, x: A) {
    function_odd(f) implies f(-x) = -f(x)
} by {
    if function_odd(f) {
        function_odd_def(f)
        forall(y: A) {
            f(-y) = -f(y)
        }
        f(-x) = -f(x)
    }
}

/// Every constant function is even.
theorem function_even_const[A: Neg, B](b: B) {
    function_even(constant[A, B](b))
} by {
    forall(x: A) {
        constant[A, B](b, -x) = b
        constant[A, B](b, x) = b
        constant[A, B](b, -x) = constant[A, B](b, x)
    }
}

/// The zero function is even.
theorem function_even_zero[A: Neg, B: Zero] {
    function_even(pointwise_zero[A, B])
} by {
    forall(x: A) {
        pointwise_zero[A, B](-x) = B.0
        pointwise_zero[A, B](x) = B.0
        pointwise_zero[A, B](-x) = pointwise_zero[A, B](x)
    }
}

/// The zero function is odd when the codomain is an additive group.
theorem function_odd_zero[A: Neg, B: AddGroup] {
    function_odd(pointwise_zero[A, B])
} by {
    forall(x: A) {
        pointwise_zero[A, B](-x) = B.0
        pointwise_zero[A, B](x) = B.0
        -B.0 = B.0
        pointwise_zero[A, B](-x) = -pointwise_zero[A, B](x)
    }
}

/// Composing any function after an even function gives an even function.
theorem function_even_left_comp[A: Neg, B, C](g: A -> B, f: B -> C) {
    function_even(g) implies function_even(compose(f, g))
} by {
    if function_even(g) {
        forall(x: A) {
            compose(f, g, -x) = f(g(-x))
            function_even_eq(g, x)
            g(-x) = g(x)
            compose(f, g, -x) = f(g(x))
            compose(f, g, x) = f(g(x))
            compose(f, g, -x) = compose(f, g, x)
        }
    }
}

/// Composing an even function after an odd function gives an even function.
theorem function_even_comp_odd[A: Neg, B: Neg, C](f: B -> C, g: A -> B) {
    function_even(f) and function_odd(g) implies function_even(compose(f, g))
} by {
    if function_even(f) and function_odd(g) {
        forall(x: A) {
            compose(f, g, -x) = f(g(-x))
            function_odd_eq(g, x)
            g(-x) = -g(x)
            compose(f, g, -x) = f(-g(x))
            function_even_eq(f, g(x))
            f(-g(x)) = f(g(x))
            compose(f, g, x) = f(g(x))
            compose(f, g, -x) = compose(f, g, x)
        }
    }
}

/// Composing an odd function after an odd function gives an odd function.
theorem function_odd_comp_odd[A: Neg, B: Neg, C: Neg](f: B -> C, g: A -> B) {
    function_odd(f) and function_odd(g) implies function_odd(compose(f, g))
} by {
    if function_odd(f) and function_odd(g) {
        forall(x: A) {
            compose(f, g, -x) = f(g(-x))
            function_odd_eq(g, x)
            g(-x) = -g(x)
            compose(f, g, -x) = f(-g(x))
            function_odd_eq(f, g(x))
            f(-g(x)) = -f(g(x))
            compose(f, g, x) = f(g(x))
            compose(f, g, -x) = -compose(f, g, x)
        }
    }
}

/// The pointwise sum of two even functions is even.
theorem function_even_add[A: Neg, B: Add](f: A -> B, g: A -> B) {
    function_even(f) and function_even(g) implies function_even(pointwise_add(f, g))
} by {
    if function_even(f) and function_even(g) {
        forall(x: A) {
            pointwise_add(f, g, -x) = f(-x) + g(-x)
            function_even_eq(f, x)
            function_even_eq(g, x)
            f(-x) = f(x)
            g(-x) = g(x)
            pointwise_add(f, g, -x) = f(x) + g(x)
            pointwise_add(f, g, x) = f(x) + g(x)
            pointwise_add(f, g, -x) = pointwise_add(f, g, x)
        }
    }
}

/// The pointwise sum of two odd functions is odd.
theorem function_odd_add[A: Neg, B: AddCommGroup](f: A -> B, g: A -> B) {
    function_odd(f) and function_odd(g) implies function_odd(pointwise_add(f, g))
} by {
    if function_odd(f) and function_odd(g) {
        forall(x: A) {
            pointwise_add(f, g, -x) = f(-x) + g(-x)
            function_odd_eq(f, x)
            function_odd_eq(g, x)
            f(-x) = -f(x)
            g(-x) = -g(x)
            pointwise_add(f, g, -x) = -f(x) + -g(x)
            pointwise_add(f, g, x) = f(x) + g(x)
            inverse_add(f(x), g(x))
            -(f(x) + g(x)) = -g(x) + -f(x)
            -g(x) + -f(x) = -f(x) + -g(x)
            -(f(x) + g(x)) = -f(x) + -g(x)
            -pointwise_add(f, g, x) = -(f(x) + g(x))
            -pointwise_add(f, g, x) = -f(x) + -g(x)
            pointwise_add(f, g, -x) = -pointwise_add(f, g, x)
        }
    }
}

/// The pointwise product of two even functions is even.
theorem function_even_mul_even[A: Neg, B: Mul](f: A -> B, g: A -> B) {
    function_even(f) and function_even(g) implies function_even(pointwise_mul(f, g))
} by {
    if function_even(f) and function_even(g) {
        forall(x: A) {
            pointwise_mul(f, g, -x) = f(-x) * g(-x)
            function_even_eq(f, x)
            function_even_eq(g, x)
            f(-x) = f(x)
            g(-x) = g(x)
            pointwise_mul(f, g, -x) = f(x) * g(x)
            pointwise_mul(f, g, x) = f(x) * g(x)
            pointwise_mul(f, g, -x) = pointwise_mul(f, g, x)
        }
    }
}
