/// Units for pointwise algebraic operations on binary products of function spaces.

from pair import Pair, pair_ext
from algebra.monoid.monoid import Monoid
from algebra.group import Group
from data.basic.function_units import is_pointwise_unit, pointwise_one_is_pointwise_unit,
    pointwise_mul_is_pointwise_unit, pointwise_unit_of_eq_reverse, pointwise_unit_of_group
from data.basic.function_algebra import pointwise_mul, pointwise_one
from data.basic.function_product_algebra import function_pair_mul, function_pair_one,
    function_pair_mul_first, function_pair_mul_second, function_pair_inverse,
    function_pair_mul_inverse_right, function_pair_mul_inverse_left,
    function_triple, function_triple_mul, function_triple_one, function_triple_inverse,
    function_triple_first, function_triple_second_first, function_triple_second_second,
    function_triple_ext, function_triple_mul_inverse_right, function_triple_mul_inverse_left,
    function_quadruple_mul, function_quadruple_one, function_quadruple_inverse,
    function_quadruple_mul_inverse_right, function_quadruple_mul_inverse_left

/// True if a binary product of functions has a two-sided pointwise inverse.
define is_function_pair_unit[T, A: Monoid, B: Monoid](p: Pair[T -> A, T -> B]) -> Bool {
    exists(q: Pair[T -> A, T -> B]) {
        function_pair_mul(p, q) = function_pair_one[T, A, B] and
        function_pair_mul(q, p) = function_pair_one[T, A, B]
    }
}

/// A function-pair unit has a pointwise-unit first coordinate.
theorem function_pair_unit_first_is_pointwise_unit[T, A: Monoid, B: Monoid](
    p: Pair[T -> A, T -> B]
) {
    is_function_pair_unit(p) implies is_pointwise_unit(p.first)
} by {
    if is_function_pair_unit(p) {
        let q: Pair[T -> A, T -> B] satisfy {
            function_pair_mul(p, q) = function_pair_one[T, A, B] and
            function_pair_mul(q, p) = function_pair_one[T, A, B]
        }
        function_pair_mul(p, q).first = pointwise_mul(p.first, q.first)
        function_pair_one[T, A, B].first = pointwise_one[T, A]
        pointwise_mul(p.first, q.first) = pointwise_one[T, A]
        function_pair_mul(q, p).first = pointwise_mul(q.first, p.first)
        pointwise_mul(q.first, p.first) = pointwise_one[T, A]
        exists(f: T -> A) {
            f = q.first and
            pointwise_mul(p.first, f) = pointwise_one[T, A] and
            pointwise_mul(f, p.first) = pointwise_one[T, A]
        }
    }
}

/// A function-pair unit has a pointwise-unit second coordinate.
theorem function_pair_unit_second_is_pointwise_unit[T, A: Monoid, B: Monoid](
    p: Pair[T -> A, T -> B]
) {
    is_function_pair_unit(p) implies is_pointwise_unit(p.second)
} by {
    if is_function_pair_unit(p) {
        let q: Pair[T -> A, T -> B] satisfy {
            function_pair_mul(p, q) = function_pair_one[T, A, B] and
            function_pair_mul(q, p) = function_pair_one[T, A, B]
        }
        function_pair_mul(p, q).second = pointwise_mul(p.second, q.second)
        function_pair_one[T, A, B].second = pointwise_one[T, B]
        pointwise_mul(p.second, q.second) = pointwise_one[T, B]
        function_pair_mul(q, p).second = pointwise_mul(q.second, p.second)
        pointwise_mul(q.second, p.second) = pointwise_one[T, B]
        exists(g: T -> B) {
            g = q.second and
            pointwise_mul(p.second, g) = pointwise_one[T, B] and
            pointwise_mul(g, p.second) = pointwise_one[T, B]
        }
    }
}

/// A binary product of pointwise-unit functions is a function-pair unit.
theorem function_pair_unit_of_components[T, A: Monoid, B: Monoid](
    p: Pair[T -> A, T -> B]
) {
    is_pointwise_unit(p.first) and is_pointwise_unit(p.second) implies
    is_function_pair_unit(p)
} by {
    if is_pointwise_unit(p.first) and is_pointwise_unit(p.second) {
        let f: T -> A satisfy {
            pointwise_mul(p.first, f) = pointwise_one[T, A] and
            pointwise_mul(f, p.first) = pointwise_one[T, A]
        }
        let g: T -> B satisfy {
            pointwise_mul(p.second, g) = pointwise_one[T, B] and
            pointwise_mul(g, p.second) = pointwise_one[T, B]
        }
        let q = Pair.new(f, g)
        let rhs = function_pair_one[T, A, B]
        let left = function_pair_mul(p, q)
        left.first = pointwise_mul(p.first, q.first)
        q.first = f
        left.first = pointwise_mul(p.first, f)
        left.first = pointwise_one[T, A]
        rhs.first = pointwise_one[T, A]
        left.first = rhs.first
        left.second = pointwise_mul(p.second, q.second)
        q.second = g
        left.second = pointwise_mul(p.second, g)
        left.second = pointwise_one[T, B]
        rhs.second = pointwise_one[T, B]
        left.second = rhs.second
        pair_ext(left, rhs)

        let right = function_pair_mul(q, p)
        right.first = pointwise_mul(q.first, p.first)
        right.first = pointwise_mul(f, p.first)
        right.first = pointwise_one[T, A]
        right.first = rhs.first
        right.second = pointwise_mul(q.second, p.second)
        right.second = pointwise_mul(g, p.second)
        right.second = pointwise_one[T, B]
        right.second = rhs.second
        pair_ext(right, rhs)

        function_pair_mul(p, q) = function_pair_one[T, A, B]
        function_pair_mul(q, p) = function_pair_one[T, A, B]
        exists(r: Pair[T -> A, T -> B]) {
            r = q and
            function_pair_mul(p, r) = function_pair_one[T, A, B] and
            function_pair_mul(r, p) = function_pair_one[T, A, B]
        }
    }
}

/// A function pair is a unit exactly when both coordinates are pointwise units.
theorem function_pair_unit_iff_components[T, A: Monoid, B: Monoid](
    p: Pair[T -> A, T -> B]
) {
    is_function_pair_unit(p) = (is_pointwise_unit(p.first) and is_pointwise_unit(p.second))
} by {
    if is_function_pair_unit(p) {
        function_pair_unit_first_is_pointwise_unit(p)
        function_pair_unit_second_is_pointwise_unit(p)
        is_pointwise_unit(p.first) and is_pointwise_unit(p.second)
    }
    if is_pointwise_unit(p.first) and is_pointwise_unit(p.second) {
        function_pair_unit_of_components(p)
        is_function_pair_unit(p)
    }
}

/// Pointwise-unit coordinates give a pointwise-unit first coordinate after function-pair multiplication.
theorem function_pair_mul_first_is_pointwise_unit[T, A: Monoid, B: Monoid](
    p: Pair[T -> A, T -> B],
    q: Pair[T -> A, T -> B]
) {
    is_pointwise_unit(p.first) and is_pointwise_unit(q.first) implies
    is_pointwise_unit(function_pair_mul(p, q).first)
} by {
    if is_pointwise_unit(p.first) and is_pointwise_unit(q.first) {
        pointwise_mul_is_pointwise_unit(p.first, q.first)
        function_pair_mul_first(p, q)
        is_pointwise_unit(pointwise_mul(p.first, q.first)) and
            function_pair_mul(p, q).first = pointwise_mul(p.first, q.first)
        pointwise_unit_of_eq_reverse(function_pair_mul(p, q).first, pointwise_mul(p.first, q.first))
        is_pointwise_unit(function_pair_mul(p, q).first)
    }
}

/// Pointwise-unit coordinates give a pointwise-unit second coordinate after function-pair multiplication.
theorem function_pair_mul_second_is_pointwise_unit[T, A: Monoid, B: Monoid](
    p: Pair[T -> A, T -> B],
    q: Pair[T -> A, T -> B]
) {
    is_pointwise_unit(p.second) and is_pointwise_unit(q.second) implies
    is_pointwise_unit(function_pair_mul(p, q).second)
} by {
    if is_pointwise_unit(p.second) and is_pointwise_unit(q.second) {
        pointwise_mul_is_pointwise_unit(p.second, q.second)
        function_pair_mul_second(p, q)
        is_pointwise_unit(pointwise_mul(p.second, q.second)) and
            function_pair_mul(p, q).second = pointwise_mul(p.second, q.second)
        pointwise_unit_of_eq_reverse(function_pair_mul(p, q).second, pointwise_mul(p.second, q.second))
        is_pointwise_unit(function_pair_mul(p, q).second)
    }
}

/// The componentwise product of two function-pair units is a function-pair unit.
theorem function_pair_mul_is_function_pair_unit[T, A: Monoid, B: Monoid](
    p: Pair[T -> A, T -> B],
    q: Pair[T -> A, T -> B]
) {
    is_function_pair_unit(p) and is_function_pair_unit(q) implies
    is_function_pair_unit(function_pair_mul(p, q))
} by {
    if is_function_pair_unit(p) and is_function_pair_unit(q) {
        function_pair_unit_first_is_pointwise_unit(p)
        function_pair_unit_first_is_pointwise_unit(q)
        is_pointwise_unit(p.first) and is_pointwise_unit(q.first)
        function_pair_mul_first_is_pointwise_unit(p, q)
        is_pointwise_unit(function_pair_mul(p, q).first)

        function_pair_unit_second_is_pointwise_unit(p)
        function_pair_unit_second_is_pointwise_unit(q)
        is_pointwise_unit(p.second) and is_pointwise_unit(q.second)
        function_pair_mul_second_is_pointwise_unit(p, q)
        is_pointwise_unit(function_pair_mul(p, q).second)

        is_pointwise_unit(function_pair_mul(p, q).first) and
            is_pointwise_unit(function_pair_mul(p, q).second)
        function_pair_unit_of_components(function_pair_mul(p, q))
        is_function_pair_unit(function_pair_mul(p, q))
    }
}

/// The pointwise identity function pair is a function-pair unit.
theorem function_pair_one_is_function_pair_unit[T, A: Monoid, B: Monoid] {
    is_function_pair_unit(function_pair_one[T, A, B])
} by {
    pointwise_one_is_pointwise_unit[T, A]
    pointwise_one_is_pointwise_unit[T, B]
    function_pair_one[T, A, B].first = pointwise_one[T, A]
    function_pair_one[T, A, B].second = pointwise_one[T, B]
    is_pointwise_unit(function_pair_one[T, A, B].first)
    is_pointwise_unit(function_pair_one[T, A, B].second)
    function_pair_unit_of_components(function_pair_one[T, A, B])
}

/// A stated two-sided pointwise inverse makes the original function pair a unit.
theorem function_pair_unit_of_inverse[T, A: Monoid, B: Monoid](
    p: Pair[T -> A, T -> B],
    q: Pair[T -> A, T -> B]
) {
    function_pair_mul(p, q) = function_pair_one[T, A, B] and
    function_pair_mul(q, p) = function_pair_one[T, A, B]
    implies is_function_pair_unit(p)
} by {
    if function_pair_mul(p, q) = function_pair_one[T, A, B] and
        function_pair_mul(q, p) = function_pair_one[T, A, B] {
        exists(r: Pair[T -> A, T -> B]) {
            r = q and
            function_pair_mul(p, r) = function_pair_one[T, A, B] and
            function_pair_mul(r, p) = function_pair_one[T, A, B]
        }
    }
}

/// A stated two-sided pointwise inverse is itself a function-pair unit.
theorem function_pair_inverse_witness_is_function_pair_unit[T, A: Monoid, B: Monoid](
    p: Pair[T -> A, T -> B],
    q: Pair[T -> A, T -> B]
) {
    function_pair_mul(p, q) = function_pair_one[T, A, B] and
    function_pair_mul(q, p) = function_pair_one[T, A, B]
    implies is_function_pair_unit(q)
} by {
    if function_pair_mul(p, q) = function_pair_one[T, A, B] and
        function_pair_mul(q, p) = function_pair_one[T, A, B] {
        exists(r: Pair[T -> A, T -> B]) {
            r = p and
            function_pair_mul(q, r) = function_pair_one[T, A, B] and
            function_pair_mul(r, q) = function_pair_one[T, A, B]
        }
    }
}

/// Every binary product of functions into groups is a function-pair unit.
theorem function_pair_unit_of_group[T, A: Group, B: Group](p: Pair[T -> A, T -> B]) {
    is_function_pair_unit(p)
} by {
    pointwise_unit_of_group(p.first)
    pointwise_unit_of_group(p.second)
    function_pair_unit_of_components(p)
}

/// The pointwise inverse of a group-valued function pair is a function-pair unit.
theorem function_pair_inverse_is_function_pair_unit[T, A: Group, B: Group](
    p: Pair[T -> A, T -> B]
) {
    is_function_pair_unit(function_pair_inverse(p))
} by {
    function_pair_mul_inverse_right(p)
    function_pair_mul_inverse_left(p)
    function_pair_inverse_witness_is_function_pair_unit(p, function_pair_inverse(p))
}

/// True if a ternary product of functions has a two-sided pointwise inverse.
define is_function_triple_unit[T, A: Monoid, B: Monoid, C: Monoid](
    p: Pair[T -> A, Pair[T -> B, T -> C]]
) -> Bool {
    exists(q: Pair[T -> A, Pair[T -> B, T -> C]]) {
        function_triple_mul(p, q) = function_triple_one[T, A, B, C] and
        function_triple_mul(q, p) = function_triple_one[T, A, B, C]
    }
}

/// A function-triple unit has a pointwise-unit first coordinate.
theorem function_triple_unit_first_is_pointwise_unit[T, A: Monoid, B: Monoid, C: Monoid](
    p: Pair[T -> A, Pair[T -> B, T -> C]]
) {
    is_function_triple_unit(p) implies is_pointwise_unit(p.first)
} by {
    if is_function_triple_unit(p) {
        let q: Pair[T -> A, Pair[T -> B, T -> C]] satisfy {
            function_triple_mul(p, q) = function_triple_one[T, A, B, C] and
            function_triple_mul(q, p) = function_triple_one[T, A, B, C]
        }
        function_triple_mul(p, q).first = pointwise_mul(p.first, q.first)
        function_triple_one[T, A, B, C].first = pointwise_one[T, A]
        pointwise_mul(p.first, q.first) = pointwise_one[T, A]
        function_triple_mul(q, p).first = pointwise_mul(q.first, p.first)
        pointwise_mul(q.first, p.first) = pointwise_one[T, A]
        exists(f: T -> A) {
            f = q.first and
            pointwise_mul(p.first, f) = pointwise_one[T, A] and
            pointwise_mul(f, p.first) = pointwise_one[T, A]
        }
    }
}

/// A function-triple unit has a pointwise-unit second coordinate.
theorem function_triple_unit_second_first_is_pointwise_unit[T, A: Monoid, B: Monoid, C: Monoid](
    p: Pair[T -> A, Pair[T -> B, T -> C]]
) {
    is_function_triple_unit(p) implies is_pointwise_unit(p.second.first)
} by {
    if is_function_triple_unit(p) {
        let q: Pair[T -> A, Pair[T -> B, T -> C]] satisfy {
            function_triple_mul(p, q) = function_triple_one[T, A, B, C] and
            function_triple_mul(q, p) = function_triple_one[T, A, B, C]
        }
        function_triple_mul(p, q).second.first = pointwise_mul(p.second.first, q.second.first)
        function_triple_one[T, A, B, C].second.first = pointwise_one[T, B]
        pointwise_mul(p.second.first, q.second.first) = pointwise_one[T, B]
        function_triple_mul(q, p).second.first = pointwise_mul(q.second.first, p.second.first)
        pointwise_mul(q.second.first, p.second.first) = pointwise_one[T, B]
        exists(g: T -> B) {
            g = q.second.first and
            pointwise_mul(p.second.first, g) = pointwise_one[T, B] and
            pointwise_mul(g, p.second.first) = pointwise_one[T, B]
        }
    }
}

/// A function-triple unit has a pointwise-unit third coordinate.
theorem function_triple_unit_second_second_is_pointwise_unit[T, A: Monoid, B: Monoid, C: Monoid](
    p: Pair[T -> A, Pair[T -> B, T -> C]]
) {
    is_function_triple_unit(p) implies is_pointwise_unit(p.second.second)
} by {
    if is_function_triple_unit(p) {
        let q: Pair[T -> A, Pair[T -> B, T -> C]] satisfy {
            function_triple_mul(p, q) = function_triple_one[T, A, B, C] and
            function_triple_mul(q, p) = function_triple_one[T, A, B, C]
        }
        function_triple_mul(p, q).second.second =
            pointwise_mul(p.second.second, q.second.second)
        function_triple_one[T, A, B, C].second.second = pointwise_one[T, C]
        pointwise_mul(p.second.second, q.second.second) = pointwise_one[T, C]
        function_triple_mul(q, p).second.second =
            pointwise_mul(q.second.second, p.second.second)
        pointwise_mul(q.second.second, p.second.second) = pointwise_one[T, C]
        exists(h: T -> C) {
            h = q.second.second and
            pointwise_mul(p.second.second, h) = pointwise_one[T, C] and
            pointwise_mul(h, p.second.second) = pointwise_one[T, C]
        }
    }
}

/// A ternary product of pointwise-unit functions is a function-triple unit.
theorem function_triple_unit_of_components[T, A: Monoid, B: Monoid, C: Monoid](
    p: Pair[T -> A, Pair[T -> B, T -> C]]
) {
    is_pointwise_unit(p.first) and is_pointwise_unit(p.second.first) and
    is_pointwise_unit(p.second.second) implies is_function_triple_unit(p)
} by {
    if is_pointwise_unit(p.first) and is_pointwise_unit(p.second.first) and
    is_pointwise_unit(p.second.second) {
        let f: T -> A satisfy {
            pointwise_mul(p.first, f) = pointwise_one[T, A] and
            pointwise_mul(f, p.first) = pointwise_one[T, A]
        }
        let g: T -> B satisfy {
            pointwise_mul(p.second.first, g) = pointwise_one[T, B] and
            pointwise_mul(g, p.second.first) = pointwise_one[T, B]
        }
        let h: T -> C satisfy {
            pointwise_mul(p.second.second, h) = pointwise_one[T, C] and
            pointwise_mul(h, p.second.second) = pointwise_one[T, C]
        }
        let q = function_triple(f, g, h)
        function_triple_first(f, g, h)
        function_triple_second_first(f, g, h)
        function_triple_second_second(f, g, h)
        let rhs = function_triple_one[T, A, B, C]
        let left = function_triple_mul(p, q)
        left.first = pointwise_mul(p.first, q.first)
        q.first = f
        left.first = pointwise_mul(p.first, f)
        left.first = pointwise_one[T, A]
        rhs.first = pointwise_one[T, A]
        left.first = rhs.first
        left.second.first = pointwise_mul(p.second.first, q.second.first)
        q.second.first = g
        left.second.first = pointwise_mul(p.second.first, g)
        left.second.first = pointwise_one[T, B]
        rhs.second.first = pointwise_one[T, B]
        left.second.first = rhs.second.first
        left.second.second = pointwise_mul(p.second.second, q.second.second)
        q.second.second = h
        left.second.second = pointwise_mul(p.second.second, h)
        left.second.second = pointwise_one[T, C]
        rhs.second.second = pointwise_one[T, C]
        left.second.second = rhs.second.second
        function_triple_ext(left, rhs)

        let right = function_triple_mul(q, p)
        right.first = pointwise_mul(q.first, p.first)
        right.first = pointwise_mul(f, p.first)
        right.first = pointwise_one[T, A]
        right.first = rhs.first
        q.second = Pair.new(g, h)
        right.second = Pair.new(pointwise_mul(q.second.first, p.second.first),
            pointwise_mul(q.second.second, p.second.second))
        right.second.first = pointwise_mul(q.second.first, p.second.first)
        q.second.first = g
        right.second.first = pointwise_mul(g, p.second.first)
        right.second.first = pointwise_one[T, B]
        right.second.first = rhs.second.first
        right.second.second = pointwise_mul(q.second.second, p.second.second)
        q.second.second = h
        right.second.second = pointwise_mul(h, p.second.second)
        right.second.second = pointwise_one[T, C]
        right.second.second = rhs.second.second
        function_triple_ext(right, rhs)

        function_triple_mul(p, q) = function_triple_one[T, A, B, C]
        function_triple_mul(q, p) = function_triple_one[T, A, B, C]
        exists(r: Pair[T -> A, Pair[T -> B, T -> C]]) {
            r = q and
            function_triple_mul(p, r) = function_triple_one[T, A, B, C] and
            function_triple_mul(r, p) = function_triple_one[T, A, B, C]
        }
    }
}

/// A function triple is a unit exactly when all three coordinates are pointwise units.
theorem function_triple_unit_iff_components[T, A: Monoid, B: Monoid, C: Monoid](
    p: Pair[T -> A, Pair[T -> B, T -> C]]
) {
    is_function_triple_unit(p) = (
        is_pointwise_unit(p.first) and
        is_pointwise_unit(p.second.first) and
        is_pointwise_unit(p.second.second)
    )
} by {
    if is_function_triple_unit(p) {
        function_triple_unit_first_is_pointwise_unit(p)
        function_triple_unit_second_first_is_pointwise_unit(p)
        function_triple_unit_second_second_is_pointwise_unit(p)
        is_pointwise_unit(p.first) and
        is_pointwise_unit(p.second.first) and
        is_pointwise_unit(p.second.second)
    }
    if is_pointwise_unit(p.first) and
        is_pointwise_unit(p.second.first) and
        is_pointwise_unit(p.second.second) {
        function_triple_unit_of_components(p)
        is_function_triple_unit(p)
    }
}

/// The pointwise identity function triple is a function-triple unit.
theorem function_triple_one_is_function_triple_unit[T, A: Monoid, B: Monoid, C: Monoid] {
    is_function_triple_unit(function_triple_one[T, A, B, C])
} by {
    pointwise_one_is_pointwise_unit[T, A]
    pointwise_one_is_pointwise_unit[T, B]
    pointwise_one_is_pointwise_unit[T, C]
    function_triple_one[T, A, B, C].first = pointwise_one[T, A]
    function_triple_one[T, A, B, C].second.first = pointwise_one[T, B]
    function_triple_one[T, A, B, C].second.second = pointwise_one[T, C]
    is_pointwise_unit(function_triple_one[T, A, B, C].first)
    is_pointwise_unit(function_triple_one[T, A, B, C].second.first)
    is_pointwise_unit(function_triple_one[T, A, B, C].second.second)
    function_triple_unit_of_components(function_triple_one[T, A, B, C])
}

/// A stated two-sided pointwise inverse makes the original function triple a unit.
theorem function_triple_unit_of_inverse[T, A: Monoid, B: Monoid, C: Monoid](
    p: Pair[T -> A, Pair[T -> B, T -> C]],
    q: Pair[T -> A, Pair[T -> B, T -> C]]
) {
    function_triple_mul(p, q) = function_triple_one[T, A, B, C] and
    function_triple_mul(q, p) = function_triple_one[T, A, B, C]
    implies is_function_triple_unit(p)
} by {
    if function_triple_mul(p, q) = function_triple_one[T, A, B, C] and
        function_triple_mul(q, p) = function_triple_one[T, A, B, C] {
        exists(r: Pair[T -> A, Pair[T -> B, T -> C]]) {
            r = q and
            function_triple_mul(p, r) = function_triple_one[T, A, B, C] and
            function_triple_mul(r, p) = function_triple_one[T, A, B, C]
        }
    }
}

/// A stated two-sided pointwise inverse is itself a function-triple unit.
theorem function_triple_inverse_witness_is_function_triple_unit[T, A: Monoid, B: Monoid, C: Monoid](
    p: Pair[T -> A, Pair[T -> B, T -> C]],
    q: Pair[T -> A, Pair[T -> B, T -> C]]
) {
    function_triple_mul(p, q) = function_triple_one[T, A, B, C] and
    function_triple_mul(q, p) = function_triple_one[T, A, B, C]
    implies is_function_triple_unit(q)
} by {
    if function_triple_mul(p, q) = function_triple_one[T, A, B, C] and
        function_triple_mul(q, p) = function_triple_one[T, A, B, C] {
        exists(r: Pair[T -> A, Pair[T -> B, T -> C]]) {
            r = p and
            function_triple_mul(q, r) = function_triple_one[T, A, B, C] and
            function_triple_mul(r, q) = function_triple_one[T, A, B, C]
        }
    }
}

/// Every ternary product of functions into groups is a function-triple unit.
theorem function_triple_unit_of_group[T, A: Group, B: Group, C: Group](
    p: Pair[T -> A, Pair[T -> B, T -> C]]
) {
    is_function_triple_unit(p)
} by {
    pointwise_unit_of_group(p.first)
    pointwise_unit_of_group(p.second.first)
    pointwise_unit_of_group(p.second.second)
    function_triple_unit_of_components(p)
}

/// The pointwise inverse of a group-valued function triple is a function-triple unit.
theorem function_triple_inverse_is_function_triple_unit[T, A: Group, B: Group, C: Group](
    p: Pair[T -> A, Pair[T -> B, T -> C]]
) {
    is_function_triple_unit(function_triple_inverse(p))
} by {
    function_triple_mul_inverse_right(p)
    function_triple_mul_inverse_left(p)
    function_triple_inverse_witness_is_function_triple_unit(p, function_triple_inverse(p))
}

/// True if a quaternary product of functions has a two-sided pointwise inverse.
define is_function_quadruple_unit[T, A: Monoid, B: Monoid, C: Monoid, D: Monoid](
    p: Pair[T -> A, Pair[T -> B, Pair[T -> C, T -> D]]]
) -> Bool {
    exists(q: Pair[T -> A, Pair[T -> B, Pair[T -> C, T -> D]]]) {
        function_quadruple_mul(p, q) = function_quadruple_one[T, A, B, C, D] and
        function_quadruple_mul(q, p) = function_quadruple_one[T, A, B, C, D]
    }
}

/// A function-quadruple unit has a pointwise-unit first coordinate.
theorem function_quadruple_unit_first_is_pointwise_unit[T, A: Monoid, B: Monoid, C: Monoid, D: Monoid](
    p: Pair[T -> A, Pair[T -> B, Pair[T -> C, T -> D]]]
) {
    is_function_quadruple_unit(p) implies is_pointwise_unit(p.first)
} by {
    if is_function_quadruple_unit(p) {
        let q: Pair[T -> A, Pair[T -> B, Pair[T -> C, T -> D]]] satisfy {
            function_quadruple_mul(p, q) = function_quadruple_one[T, A, B, C, D] and
            function_quadruple_mul(q, p) = function_quadruple_one[T, A, B, C, D]
        }
        function_quadruple_mul(p, q).first = pointwise_mul(p.first, q.first)
        function_quadruple_one[T, A, B, C, D].first = pointwise_one[T, A]
        pointwise_mul(p.first, q.first) = pointwise_one[T, A]
        function_quadruple_mul(q, p).first = pointwise_mul(q.first, p.first)
        pointwise_mul(q.first, p.first) = pointwise_one[T, A]
        exists(f: T -> A) {
            f = q.first and
            pointwise_mul(p.first, f) = pointwise_one[T, A] and
            pointwise_mul(f, p.first) = pointwise_one[T, A]
        }
    }
}

/// A function-quadruple unit has a pointwise-unit second coordinate.
theorem function_quadruple_unit_second_first_is_pointwise_unit[T, A: Monoid, B: Monoid, C: Monoid, D: Monoid](
    p: Pair[T -> A, Pair[T -> B, Pair[T -> C, T -> D]]]
) {
    is_function_quadruple_unit(p) implies is_pointwise_unit(p.second.first)
} by {
    if is_function_quadruple_unit(p) {
        let q: Pair[T -> A, Pair[T -> B, Pair[T -> C, T -> D]]] satisfy {
            function_quadruple_mul(p, q) = function_quadruple_one[T, A, B, C, D] and
            function_quadruple_mul(q, p) = function_quadruple_one[T, A, B, C, D]
        }
        function_quadruple_mul(p, q).second.first =
            pointwise_mul(p.second.first, q.second.first)
        function_quadruple_one[T, A, B, C, D].second.first = pointwise_one[T, B]
        pointwise_mul(p.second.first, q.second.first) = pointwise_one[T, B]
        function_quadruple_mul(q, p).second.first =
            pointwise_mul(q.second.first, p.second.first)
        pointwise_mul(q.second.first, p.second.first) = pointwise_one[T, B]
        exists(g: T -> B) {
            g = q.second.first and
            pointwise_mul(p.second.first, g) = pointwise_one[T, B] and
            pointwise_mul(g, p.second.first) = pointwise_one[T, B]
        }
    }
}

/// A function-quadruple unit has a pointwise-unit third coordinate.
theorem function_quadruple_unit_second_second_first_is_pointwise_unit[T, A: Monoid, B: Monoid, C: Monoid, D: Monoid](
    p: Pair[T -> A, Pair[T -> B, Pair[T -> C, T -> D]]]
) {
    is_function_quadruple_unit(p) implies is_pointwise_unit(p.second.second.first)
} by {
    if is_function_quadruple_unit(p) {
        let q: Pair[T -> A, Pair[T -> B, Pair[T -> C, T -> D]]] satisfy {
            function_quadruple_mul(p, q) = function_quadruple_one[T, A, B, C, D] and
            function_quadruple_mul(q, p) = function_quadruple_one[T, A, B, C, D]
        }
        function_quadruple_mul(p, q).second.second.first =
            pointwise_mul(p.second.second.first, q.second.second.first)
        function_quadruple_one[T, A, B, C, D].second.second.first = pointwise_one[T, C]
        pointwise_mul(p.second.second.first, q.second.second.first) = pointwise_one[T, C]
        function_quadruple_mul(q, p).second.second.first =
            pointwise_mul(q.second.second.first, p.second.second.first)
        pointwise_mul(q.second.second.first, p.second.second.first) = pointwise_one[T, C]
        exists(h: T -> C) {
            h = q.second.second.first and
            pointwise_mul(p.second.second.first, h) = pointwise_one[T, C] and
            pointwise_mul(h, p.second.second.first) = pointwise_one[T, C]
        }
    }
}

/// A function-quadruple unit has a pointwise-unit fourth coordinate.
theorem function_quadruple_unit_second_second_second_is_pointwise_unit[T, A: Monoid, B: Monoid, C: Monoid, D: Monoid](
    p: Pair[T -> A, Pair[T -> B, Pair[T -> C, T -> D]]]
) {
    is_function_quadruple_unit(p) implies is_pointwise_unit(p.second.second.second)
} by {
    if is_function_quadruple_unit(p) {
        let q: Pair[T -> A, Pair[T -> B, Pair[T -> C, T -> D]]] satisfy {
            function_quadruple_mul(p, q) = function_quadruple_one[T, A, B, C, D] and
            function_quadruple_mul(q, p) = function_quadruple_one[T, A, B, C, D]
        }
        function_quadruple_mul(p, q).second.second.second =
            pointwise_mul(p.second.second.second, q.second.second.second)
        function_quadruple_one[T, A, B, C, D].second.second.second = pointwise_one[T, D]
        pointwise_mul(p.second.second.second, q.second.second.second) = pointwise_one[T, D]
        function_quadruple_mul(q, p).second.second.second =
            pointwise_mul(q.second.second.second, p.second.second.second)
        pointwise_mul(q.second.second.second, p.second.second.second) = pointwise_one[T, D]
        exists(i: T -> D) {
            i = q.second.second.second and
            pointwise_mul(p.second.second.second, i) = pointwise_one[T, D] and
            pointwise_mul(i, p.second.second.second) = pointwise_one[T, D]
        }
    }
}

/// A stated two-sided pointwise inverse makes the original function quadruple a unit.
theorem function_quadruple_unit_of_inverse[T, A: Monoid, B: Monoid, C: Monoid, D: Monoid](
    p: Pair[T -> A, Pair[T -> B, Pair[T -> C, T -> D]]],
    q: Pair[T -> A, Pair[T -> B, Pair[T -> C, T -> D]]]
) {
    function_quadruple_mul(p, q) = function_quadruple_one[T, A, B, C, D] and
    function_quadruple_mul(q, p) = function_quadruple_one[T, A, B, C, D]
    implies is_function_quadruple_unit(p)
} by {
    if function_quadruple_mul(p, q) = function_quadruple_one[T, A, B, C, D] and
        function_quadruple_mul(q, p) = function_quadruple_one[T, A, B, C, D] {
        exists(r: Pair[T -> A, Pair[T -> B, Pair[T -> C, T -> D]]]) {
            r = q and
            function_quadruple_mul(p, r) = function_quadruple_one[T, A, B, C, D] and
            function_quadruple_mul(r, p) = function_quadruple_one[T, A, B, C, D]
        }
    }
}

/// A stated two-sided pointwise inverse is itself a function-quadruple unit.
theorem function_quadruple_inverse_witness_is_function_quadruple_unit[T, A: Monoid, B: Monoid, C: Monoid, D: Monoid](
    p: Pair[T -> A, Pair[T -> B, Pair[T -> C, T -> D]]],
    q: Pair[T -> A, Pair[T -> B, Pair[T -> C, T -> D]]]
) {
    function_quadruple_mul(p, q) = function_quadruple_one[T, A, B, C, D] and
    function_quadruple_mul(q, p) = function_quadruple_one[T, A, B, C, D]
    implies is_function_quadruple_unit(q)
} by {
    if function_quadruple_mul(p, q) = function_quadruple_one[T, A, B, C, D] and
        function_quadruple_mul(q, p) = function_quadruple_one[T, A, B, C, D] {
        exists(r: Pair[T -> A, Pair[T -> B, Pair[T -> C, T -> D]]]) {
            r = p and
            function_quadruple_mul(q, r) = function_quadruple_one[T, A, B, C, D] and
            function_quadruple_mul(r, q) = function_quadruple_one[T, A, B, C, D]
        }
    }
}

/// Every quaternary product of functions into groups is a function-quadruple unit.
theorem function_quadruple_unit_of_group[T, A: Group, B: Group, C: Group, D: Group](
    p: Pair[T -> A, Pair[T -> B, Pair[T -> C, T -> D]]]
) {
    is_function_quadruple_unit(p)
} by {
    function_quadruple_mul_inverse_right(p)
    function_quadruple_mul_inverse_left(p)
    function_quadruple_unit_of_inverse(p, function_quadruple_inverse(p))
}

/// The pointwise inverse of a group-valued function quadruple is a function-quadruple unit.
theorem function_quadruple_inverse_is_function_quadruple_unit[T, A: Group, B: Group, C: Group, D: Group](
    p: Pair[T -> A, Pair[T -> B, Pair[T -> C, T -> D]]]
) {
    is_function_quadruple_unit(function_quadruple_inverse(p))
} by {
    function_quadruple_mul_inverse_right(p)
    function_quadruple_mul_inverse_left(p)
    function_quadruple_inverse_witness_is_function_quadruple_unit(p, function_quadruple_inverse(p))
}
