/// Basic constructions and predicates for functions that are independent of relation algebra.

/// Composing two functions.
define compose[T, U, V](f: U -> V, g: T -> U) -> T -> V {
    function(t: T) {
        f(g(t))
    }
}

/// The identity function on a type.
define identity_fn[T](x: T) -> T {
    x
}

/// Flip the order of arguments to a two-argument function.
define flip[T, U, V](f: (T, U) -> V, a: U, b: T) -> V {
    f(b, a)
}

/// True if a function is constant.
define is_constant[T, U](f: T -> U) -> Bool {
    exists(y: U) {
        forall(x: T) {
            f(x) = y
        }
    }
}

/// Predicate extensionality: two predicates are equal when they have the same truth value on every input.
theorem predicate_extensionality[T](p: T -> Bool, q: T -> Bool) {
    (forall(x: T) { p(x) = q(x) }) implies p = q
} by {
    if forall(x: T) { p(x) = q(x) } {
        forall(x: T) {
            p(x) = q(x)
        }
    }
}

/// Function extensionality: two functions are equal when they agree on every input.
theorem function_extensionality[T, U](f: T -> U, g: T -> U) {
    (forall(x: T) { f(x) = g(x) }) implies f = g
} by {
    if forall(x: T) { f(x) = g(x) } {
        forall(x: T) {
            f(x) = g(x)
        }
    }
}

/// Binary function extensionality: two functions are equal when they agree on every pair of inputs.
theorem binary_function_extensionality[T, U, V](f: (T, U) -> V, g: (T, U) -> V) {
    (forall(x: T, y: U) { f(x, y) = g(x, y) }) implies f = g
} by {
    if forall(x: T, y: U) { f(x, y) = g(x, y) } {
        forall(x: T) {
            forall(y: U) {
                f(x, y) = g(x, y)
            }
            function_extensionality(f(x), g(x))
        }
    }
}

/// Equal functions have equal values at each input.
theorem function_eq_apply[T, U](f: T -> U, g: T -> U, x: T) {
    f = g implies f(x) = g(x)
}

/// Equal binary functions have equal values at each pair of inputs.
theorem binary_function_eq_apply[T, U, V](f: (T, U) -> V, g: (T, U) -> V, x: T, y: U) {
    f = g implies f(x, y) = g(x, y)
}

/// Equal predicates have equal truth values at each input.
theorem predicate_eq_apply[T](p: T -> Bool, q: T -> Bool, x: T) {
    p = q implies p(x) = q(x)
}

/// Equality of functions transports predicates on functions.
theorem function_eq_transport_predicate[T, U](p: (T -> U) -> Bool, f: T -> U, g: T -> U) {
    f = g and p(f) implies p(g)
}

/// Equality of functions transports predicates on functions in the reverse direction.
theorem function_eq_transport_predicate_rev[T, U](p: (T -> U) -> Bool, f: T -> U, g: T -> U) {
    f = g and p(g) implies p(f)
}

/// Equality of predicates transports predicates on predicates.
theorem predicate_eq_transport_predicate[T](p: (T -> Bool) -> Bool, q: T -> Bool, r: T -> Bool) {
    q = r and p(q) implies p(r)
}

/// Equality of predicates transports predicates on predicates in the reverse direction.
theorem predicate_eq_transport_predicate_rev[T](p: (T -> Bool) -> Bool, q: T -> Bool, r: T -> Bool) {
    q = r and p(r) implies p(q)
}

/// Equality of binary functions transports predicates on binary functions.
theorem binary_function_eq_transport_predicate[T, U, V](p: ((T, U) -> V) -> Bool, f: (T, U) -> V, g: (T, U) -> V) {
    f = g and p(f) implies p(g)
}

/// Equality of binary functions transports predicates on binary functions in the reverse direction.
theorem binary_function_eq_transport_predicate_rev[T, U, V](p: ((T, U) -> V) -> Bool, f: (T, U) -> V, g: (T, U) -> V) {
    f = g and p(g) implies p(f)
}

/// Predicate equality transports truth forward.
theorem predicate_eq_forward[T](p: T -> Bool, q: T -> Bool, x: T) {
    p = q and p(x) implies q(x)
}

/// Predicate equality transports truth backward.
theorem predicate_eq_backward[T](p: T -> Bool, q: T -> Bool, x: T) {
    p = q and q(x) implies p(x)
}

/// Equality of elements transports predicates.
theorem eq_transport_predicate[T](p: T -> Bool, x: T, y: T) {
    x = y and p(x) implies p(y)
}

/// Equality of elements transports predicates in the reverse direction.
theorem eq_transport_predicate_rev[T](p: T -> Bool, x: T, y: T) {
    x = y and p(y) implies p(x)
}

/// Left identity for composition.
theorem compose_identity_left[T, U](f: T -> U) {
    compose(identity_fn[U], f) = f
} by {
    forall(x: T) {
        compose(identity_fn[U], f, x) = identity_fn[U](f(x))
        identity_fn[U](f(x)) = f(x)
    }
    function_extensionality(compose(identity_fn[U], f), f)
}

/// Right identity for composition.
theorem compose_identity_right[T, U](f: T -> U) {
    compose(f, identity_fn[T]) = f
} by {
    forall(x: T) {
        compose(f, identity_fn[T], x) = f(identity_fn[T](x))
        identity_fn[T](x) = x
        f(identity_fn[T](x)) = f(x)
    }
    function_extensionality(compose(f, identity_fn[T]), f)
}

/// Composition is associative.
theorem compose_assoc[A, B, C, D](f: C -> D, g: B -> C, h: A -> B) {
    compose(f, compose(g, h)) = compose(compose(f, g), h)
} by {
    forall(x: A) {
        compose(f, compose(g, h), x) = f(compose(g, h)(x))
        compose(g, h, x) = g(h(x))
        compose(f, compose(g, h), x) = f(g(h(x)))
        compose(compose(f, g), h, x) = compose(f, g)(h(x))
        compose(f, g, h(x)) = f(g(h(x)))
        compose(compose(f, g), h, x) = f(g(h(x)))
        compose(f, compose(g, h), x) = compose(compose(f, g), h, x)
    }
    function_extensionality(compose(f, compose(g, h)), compose(compose(f, g), h))
}

/// Equal left factors give equal composites.
theorem compose_eq_left[T, U, V](f: U -> V, g: U -> V, h: T -> U) {
    f = g implies compose(f, h) = compose(g, h)
} by {
    if f = g {
        forall(x: T) {
            compose(f, h, x) = f(h(x))
            compose(g, h, x) = g(h(x))
            f(h(x)) = g(h(x))
            compose(f, h, x) = compose(g, h, x)
        }
        function_extensionality(compose(f, h), compose(g, h))
    }
}

/// Equal right factors give equal composites.
theorem compose_eq_right[T, U, V](f: U -> V, g: T -> U, h: T -> U) {
    g = h implies compose(f, g) = compose(f, h)
} by {
    if g = h {
        forall(x: T) {
            compose(f, g, x) = f(g(x))
            compose(f, h, x) = f(h(x))
            g(x) = h(x)
            f(g(x)) = f(h(x))
            compose(f, g, x) = compose(f, h, x)
        }
        function_extensionality(compose(f, g), compose(f, h))
    }
}

/// Equal factors give equal composites.
theorem compose_eq[T, U, V](f: U -> V, g: U -> V, h: T -> U, k: T -> U) {
    f = g and h = k implies compose(f, h) = compose(g, k)
} by {
    if f = g and h = k {
        compose_eq_left(f, g, h)
        compose(f, h) = compose(g, h)
        compose_eq_right(g, h, k)
        compose(g, h) = compose(g, k)
        compose(f, h) = compose(g, k)
    }
}

/// Composing with a constant function yields a constant function.
theorem compose_constant[T, U, V](f: U -> V, u: U) {
    compose(f, constant[T, U](u)) = constant[T, V](f(u))
} by {
    forall(x: T) {
        compose(f, constant[T, U](u), x) = f(constant[T, U](u)(x))
        constant[T, U](u)(x) = u
        f(constant[T, U](u)(x)) = f(u)
        constant[T, V](f(u), x) = f(u)
        compose(f, constant[T, U](u), x) = constant[T, V](f(u), x)
    }
    function_extensionality(compose(f, constant[T, U](u)), constant[T, V](f(u)))
}

/// Every constant function is constant.
theorem constant_is_constant[T, U](u: U) {
    is_constant(constant[T, U](u))
}

/// Equality of functions transports constancy.
theorem function_eq_transport_constant[T, U](f: T -> U, g: T -> U) {
    f = g and is_constant(f) implies is_constant(g)
} by {
    if f = g and is_constant(f) {
        function_eq_transport_predicate(is_constant[T, U], f, g)
        is_constant(g)
    }
}

/// Equality of functions transports constancy in the reverse direction.
theorem function_eq_transport_constant_rev[T, U](f: T -> U, g: T -> U) {
    f = g and is_constant(g) implies is_constant(f)
} by {
    if f = g and is_constant(g) {
        function_eq_transport_predicate_rev(is_constant[T, U], f, g)
        is_constant(f)
    }
}

/// A type with a distinguished default element.
typeclass I: Inhabited {
    /// A distinguished element of the type.
    default: I
}

/// On an inhabited domain, every constant-valued function is equal to a constant function.
theorem is_constant_imp_eq_constant[T: Inhabited, U](f: T -> U) {
    is_constant(f) implies exists(u: U) {
        constant[T, U](u) = f
    }
} by {
    let t: T satisfy {
        true
    }
    let u = f(t)
    forall(x: T) {
        constant[T, U](u)(x) = f(x)
    }
}

/// True if `f` is injective from `T` to `U`.
define is_injective_fn[T, U](f: T -> U) -> Bool {
    forall(x: T, y: T) {
        f(x) = f(y) implies x = y
    }
}

/// True if `f` is surjective from `T` onto `U`.
define is_surjective_fn[T, U](f: T -> U) -> Bool {
    forall(y: U) {
        exists(x: T) {
            f(x) = y
        }
    }
}

/// True if `f` is bijective from `T` to `U`.
define is_bijection_fn[T, U](f: T -> U) -> Bool {
    is_injective_fn(f) and is_surjective_fn(f)
}

/// A bijection is injective.
theorem bijection_fn_is_injective[T, U](f: T -> U) {
    is_bijection_fn(f) implies is_injective_fn(f)
}

/// A bijection is surjective.
theorem bijection_fn_is_surjective[T, U](f: T -> U) {
    is_bijection_fn(f) implies is_surjective_fn(f)
}

/// Equality of functions transports injectivity.
theorem function_eq_transport_injective_fn[T, U](f: T -> U, g: T -> U) {
    f = g and is_injective_fn(f) implies is_injective_fn(g)
} by {
    if f = g and is_injective_fn(f) {
        function_eq_transport_predicate(is_injective_fn[T, U], f, g)
        is_injective_fn(g)
    }
}

/// Equality of functions transports injectivity in the reverse direction.
theorem function_eq_transport_injective_fn_rev[T, U](f: T -> U, g: T -> U) {
    f = g and is_injective_fn(g) implies is_injective_fn(f)
} by {
    if f = g and is_injective_fn(g) {
        function_eq_transport_predicate_rev(is_injective_fn[T, U], f, g)
        is_injective_fn(f)
    }
}

/// Equality of functions transports surjectivity.
theorem function_eq_transport_surjective_fn[T, U](f: T -> U, g: T -> U) {
    f = g and is_surjective_fn(f) implies is_surjective_fn(g)
} by {
    if f = g and is_surjective_fn(f) {
        function_eq_transport_predicate(is_surjective_fn[T, U], f, g)
        is_surjective_fn(g)
    }
}

/// Equality of functions transports surjectivity in the reverse direction.
theorem function_eq_transport_surjective_fn_rev[T, U](f: T -> U, g: T -> U) {
    f = g and is_surjective_fn(g) implies is_surjective_fn(f)
} by {
    if f = g and is_surjective_fn(g) {
        function_eq_transport_predicate_rev(is_surjective_fn[T, U], f, g)
        is_surjective_fn(f)
    }
}

/// Equality of functions transports bijectivity.
theorem function_eq_transport_bijection_fn[T, U](f: T -> U, g: T -> U) {
    f = g and is_bijection_fn(f) implies is_bijection_fn(g)
} by {
    if f = g and is_bijection_fn(f) {
        function_eq_transport_predicate(is_bijection_fn[T, U], f, g)
        is_bijection_fn(g)
    }
}

/// Equality of functions transports bijectivity in the reverse direction.
theorem function_eq_transport_bijection_fn_rev[T, U](f: T -> U, g: T -> U) {
    f = g and is_bijection_fn(g) implies is_bijection_fn(f)
} by {
    if f = g and is_bijection_fn(g) {
        function_eq_transport_predicate_rev(is_bijection_fn[T, U], f, g)
        is_bijection_fn(f)
    }
}

/// Injectivity turns equal outputs into equal inputs.
theorem injective_fn_eq[T, U](f: T -> U, x1: T, x2: T) {
    is_injective_fn(f) and f(x1) = f(x2) implies x1 = x2
} by {
    if is_injective_fn(f) and f(x1) = f(x2) {
        x1 = x2
    }
}

/// Injectivity turns distinct inputs into distinct outputs.
theorem injective_fn_ne[T, U](f: T -> U, x1: T, x2: T) {
    is_injective_fn(f) and x1 != x2 implies f(x1) != f(x2)
} by {
    if is_injective_fn(f) and x1 != x2 {
        if f(x1) = f(x2) {
            injective_fn_eq(f, x1, x2)
            x1 = x2
            false
        }
    }
}

/// A function is injective when distinct inputs always have distinct outputs.
theorem injective_fn_of_ne[T, U](f: T -> U) {
    forall(x1: T, x2: T) {
        x1 != x2 implies f(x1) != f(x2)
    }
    implies is_injective_fn(f)
} by {
    if forall(x1: T, x2: T) { x1 != x2 implies f(x1) != f(x2) } {
        forall(x1: T, x2: T) {
            if f(x1) = f(x2) {
                if x1 != x2 {
                    f(x1) != f(x2)
                    false
                }
            }
        }
        is_injective_fn(f)
    }
}

/// Injectivity is equivalent to preserving apartness of inputs.
theorem injective_fn_iff_ne[T, U](f: T -> U) {
    is_injective_fn(f) = forall(x1: T, x2: T) {
        x1 != x2 implies f(x1) != f(x2)
    }
} by {
    if is_injective_fn(f) {
        forall(x1: T, x2: T) {
            if x1 != x2 {
                injective_fn_ne(f, x1, x2)
                f(x1) != f(x2)
            }
        }
    }
    if forall(x1: T, x2: T) { x1 != x2 implies f(x1) != f(x2) } {
        injective_fn_of_ne(f)
        is_injective_fn(f)
    }
}

/// True if `g` is a left inverse of `f`.
define is_left_inverse_fn[T, U](f: T -> U, g: U -> T) -> Bool {
    forall(x: T) {
        g(f(x)) = x
    }
}

/// True if `g` is a right inverse of `f`.
define is_right_inverse_fn[T, U](f: T -> U, g: U -> T) -> Bool {
    forall(y: U) {
        f(g(y)) = y
    }
}

/// True if `g` is both a left and a right inverse of `f`.
define is_two_sided_inverse_fn[T, U](f: T -> U, g: U -> T) -> Bool {
    is_left_inverse_fn(f, g) and is_right_inverse_fn(f, g)
}

/// A left inverse undoes `f` at every point.
theorem left_inverse_fn_apply[T, U](f: T -> U, g: U -> T, x: T) {
    is_left_inverse_fn(f, g) implies g(f(x)) = x
} by {
    if is_left_inverse_fn(f, g) {
        is_left_inverse_fn(f, g) = forall(z: T) {
            g(f(z)) = z
        }
        g(f(x)) = x
    }
}

/// A right inverse undoes `f` at every point.
theorem right_inverse_fn_apply[T, U](f: T -> U, g: U -> T, y: U) {
    is_right_inverse_fn(f, g) implies f(g(y)) = y
} by {
    if is_right_inverse_fn(f, g) {
        is_right_inverse_fn(f, g) = forall(z: U) {
            f(g(z)) = z
        }
        f(g(y)) = y
    }
}

/// A two-sided inverse undoes `f` on the left at every point.
theorem two_sided_inverse_fn_left_apply[T, U](f: T -> U, g: U -> T, x: T) {
    is_two_sided_inverse_fn(f, g) implies g(f(x)) = x
} by {
    if is_two_sided_inverse_fn(f, g) {
        is_two_sided_inverse_fn(f, g) = is_left_inverse_fn(f, g) and is_right_inverse_fn(f, g)
        left_inverse_fn_apply(f, g, x)
    }
}

/// A two-sided inverse undoes `f` on the right at every point.
theorem two_sided_inverse_fn_right_apply[T, U](f: T -> U, g: U -> T, y: U) {
    is_two_sided_inverse_fn(f, g) implies f(g(y)) = y
} by {
    if is_two_sided_inverse_fn(f, g) {
        is_two_sided_inverse_fn(f, g) = is_left_inverse_fn(f, g) and is_right_inverse_fn(f, g)
        right_inverse_fn_apply(f, g, y)
    }
}

/// The map `f` has a left inverse.
define has_left_inverse_fn[T, U](f: T -> U) -> Bool {
    exists(g: U -> T) {
        is_left_inverse_fn(f, g)
    }
}

/// The map `f` has a right inverse.
define has_right_inverse_fn[T, U](f: T -> U) -> Bool {
    exists(g: U -> T) {
        is_right_inverse_fn(f, g)
    }
}

/// A left inverse makes a function injective.
theorem left_inverse_fn_imp_injective_fn[T, U](f: T -> U, g: U -> T) {
    is_left_inverse_fn(f, g) implies is_injective_fn(f)
} by {
    if is_left_inverse_fn(f, g) {
        forall(x1: T, x2: T) {
            if f(x1) = f(x2) {
                g(f(x1)) = g(f(x2))
                g(f(x1)) = x1
                g(f(x2)) = x2
                x1 = x2
            }
        }
        is_injective_fn(f)
    }
}

/// A function with a left inverse is injective.
theorem has_left_inverse_fn_imp_injective_fn[T, U](f: T -> U) {
    has_left_inverse_fn(f) implies is_injective_fn(f)
} by {
    if has_left_inverse_fn(f) {
        let g: U -> T satisfy {
            is_left_inverse_fn(f, g)
        }
        left_inverse_fn_imp_injective_fn(f, g)
        is_injective_fn(f)
    }
}

/// A right inverse makes a function surjective.
theorem right_inverse_fn_imp_surjective_fn[T, U](f: T -> U, g: U -> T) {
    is_right_inverse_fn(f, g) implies is_surjective_fn(f)
} by {
    if is_right_inverse_fn(f, g) {
        forall(y: U) {
            f(g(y)) = y
            exists(x: T) {
                f(x) = y
            }
        }
        is_surjective_fn(f)
    }
}

/// A function with a right inverse is surjective.
theorem has_right_inverse_fn_imp_surjective_fn[T, U](f: T -> U) {
    has_right_inverse_fn(f) implies is_surjective_fn(f)
} by {
    if has_right_inverse_fn(f) {
        let g: U -> T satisfy {
            is_right_inverse_fn(f, g)
        }
        right_inverse_fn_imp_surjective_fn(f, g)
        is_surjective_fn(f)
    }
}

/// A right inverse of an injective map is also a left inverse.
theorem injective_fn_right_inverse_fn_imp_left_inverse_fn[T, U](f: T -> U, g: U -> T) {
    is_injective_fn(f) and is_right_inverse_fn(f, g) implies is_left_inverse_fn(f, g)
} by {
    if is_injective_fn(f) and is_right_inverse_fn(f, g) {
        forall(x: T) {
            f(g(f(x))) = f(x)
            injective_fn_eq(f, g(f(x)), x)
            g(f(x)) = x
        }
        is_left_inverse_fn(f, g)
    }
}

/// A two-sided inverse makes a function bijective.
theorem two_sided_inverse_fn_imp_bijection_fn[T, U](f: T -> U, g: U -> T) {
    is_two_sided_inverse_fn(f, g) implies is_bijection_fn(f)
} by {
    if is_two_sided_inverse_fn(f, g) {
        is_left_inverse_fn(f, g)
        left_inverse_fn_imp_injective_fn(f, g)
        is_injective_fn(f)
        is_right_inverse_fn(f, g)
        right_inverse_fn_imp_surjective_fn(f, g)
        is_surjective_fn(f)
        is_bijection_fn(f)
    }
}

/// Left inverse witnesses compose in the reverse order.
theorem compose_left_inverse_fn[T, U, V](f: T -> U, g: U -> V, invf: U -> T, invg: V -> U) {
    is_left_inverse_fn(f, invf) and is_left_inverse_fn(g, invg)
    implies is_left_inverse_fn(compose(g, f), compose(invf, invg))
} by {
    if is_left_inverse_fn(f, invf) and is_left_inverse_fn(g, invg) {
        forall(x: T) {
            compose(invf, invg, compose(g, f)(x)) = invf(invg(compose(g, f)(x)))
            compose(g, f, x) = g(f(x))
            invg(g(f(x))) = f(x)
            invf(invg(compose(g, f)(x))) = invf(f(x))
            invf(f(x)) = x
            compose(invf, invg, compose(g, f)(x)) = x
        }
        is_left_inverse_fn(compose(g, f), compose(invf, invg))
    }
}

/// Right inverse witnesses compose in the reverse order.
theorem compose_right_inverse_fn[T, U, V](f: T -> U, g: U -> V, invf: U -> T, invg: V -> U) {
    is_right_inverse_fn(f, invf) and is_right_inverse_fn(g, invg)
    implies is_right_inverse_fn(compose(g, f), compose(invf, invg))
} by {
    if is_right_inverse_fn(f, invf) and is_right_inverse_fn(g, invg) {
        forall(z: V) {
            compose(g, f, compose(invf, invg)(z)) = g(f(compose(invf, invg)(z)))
            compose(invf, invg, z) = invf(invg(z))
            f(invf(invg(z))) = invg(z)
            g(f(compose(invf, invg)(z))) = g(invg(z))
            g(invg(z)) = z
            compose(g, f, compose(invf, invg)(z)) = z
        }
        is_right_inverse_fn(compose(g, f), compose(invf, invg))
    }
}

/// Two-sided inverse witnesses compose in the reverse order.
theorem compose_two_sided_inverse_fn[T, U, V](f: T -> U, g: U -> V, invf: U -> T, invg: V -> U) {
    is_two_sided_inverse_fn(f, invf) and is_two_sided_inverse_fn(g, invg)
    implies is_two_sided_inverse_fn(compose(g, f), compose(invf, invg))
} by {
    if is_two_sided_inverse_fn(f, invf) and is_two_sided_inverse_fn(g, invg) {
        is_left_inverse_fn(f, invf)
        is_left_inverse_fn(g, invg)
        compose_left_inverse_fn(f, g, invf, invg)
        is_left_inverse_fn(compose(g, f), compose(invf, invg))

        is_right_inverse_fn(f, invf)
        is_right_inverse_fn(g, invg)
        compose_right_inverse_fn(f, g, invf, invg)
        is_right_inverse_fn(compose(g, f), compose(invf, invg))

        is_two_sided_inverse_fn(compose(g, f), compose(invf, invg))
    }
}

/// A two-sided inverse is symmetric.
theorem two_sided_inverse_fn_symm[T, U](f: T -> U, g: U -> T) {
    is_two_sided_inverse_fn(f, g) implies is_two_sided_inverse_fn(g, f)
} by {
    if is_two_sided_inverse_fn(f, g) {
        forall(y: U) {
            f(g(y)) = y
        }
        is_left_inverse_fn(g, f)

        forall(x: T) {
            g(f(x)) = x
        }
        is_right_inverse_fn(g, f)

        is_two_sided_inverse_fn(g, f)
    }
}

/// Two two-sided inverses of the same function are equal.
theorem two_sided_inverse_fn_unique[T, U](f: T -> U, g1: U -> T, g2: U -> T) {
    is_two_sided_inverse_fn(f, g1) and is_two_sided_inverse_fn(f, g2) implies g1 = g2
} by {
    if is_two_sided_inverse_fn(f, g1) and is_two_sided_inverse_fn(f, g2) {
        forall(y: U) {
            f(g2(y)) = y
            g1(f(g2(y))) = g2(y)
            g1(f(g2(y))) = g1(y)
            g1(y) = g2(y)
        }
        function_extensionality(g1, g2)
        g1 = g2
    }
}

/// A left inverse and a right inverse of the same function are equal.
theorem left_inverse_fn_eq_right_inverse_fn[T, U](f: T -> U, g: U -> T, h: U -> T) {
    is_left_inverse_fn(f, g) and is_right_inverse_fn(f, h) implies g = h
} by {
    if is_left_inverse_fn(f, g) and is_right_inverse_fn(f, h) {
        forall(y: U) {
            right_inverse_fn_apply(f, h, y)
            f(h(y)) = y
            left_inverse_fn_apply(f, g, h(y))
            g(f(h(y))) = h(y)
            g(y) = h(y)
        }
        function_extensionality(g, h)
        g = h
    }
}

/// Right inverses of an injective function are unique.
theorem right_inverse_fn_unique_of_injective_fn[T, U](f: T -> U, g1: U -> T, g2: U -> T) {
    is_injective_fn(f) and is_right_inverse_fn(f, g1) and is_right_inverse_fn(f, g2) implies g1 = g2
} by {
    if is_injective_fn(f) and is_right_inverse_fn(f, g1) and is_right_inverse_fn(f, g2) {
        forall(y: U) {
            right_inverse_fn_apply(f, g1, y)
            right_inverse_fn_apply(f, g2, y)
            f(g1(y)) = y
            f(g2(y)) = y
            f(g1(y)) = f(g2(y))
            injective_fn_eq(f, g1(y), g2(y))
            g1(y) = g2(y)
        }
        function_extensionality(g1, g2)
        g1 = g2
    }
}

/// Left inverses of a surjective function are unique.
theorem left_inverse_fn_unique_of_surjective_fn[T, U](f: T -> U, g1: U -> T, g2: U -> T) {
    is_surjective_fn(f) and is_left_inverse_fn(f, g1) and is_left_inverse_fn(f, g2) implies g1 = g2
} by {
    if is_surjective_fn(f) and is_left_inverse_fn(f, g1) and is_left_inverse_fn(f, g2) {
        forall(y: U) {
            let x: T satisfy {
                f(x) = y
            }
            left_inverse_fn_apply(f, g1, x)
            left_inverse_fn_apply(f, g2, x)
            g1(f(x)) = x
            g2(f(x)) = x
            g1(y) = x
            g2(y) = x
            g1(y) = g2(y)
        }
        function_extensionality(g1, g2)
        g1 = g2
    }
}

/// A two-sided inverse is itself a bijection.
theorem inverse_of_two_sided_inverse_fn_is_bijection_fn[T, U](f: T -> U, g: U -> T) {
    is_two_sided_inverse_fn(f, g) implies is_bijection_fn(g)
} by {
    if is_two_sided_inverse_fn(f, g) {
        two_sided_inverse_fn_symm(f, g)
        is_two_sided_inverse_fn(g, f)
        two_sided_inverse_fn_imp_bijection_fn(g, f)
        is_bijection_fn(g)
    }
}

/// Composition preserves injectivity.
theorem compose_injective_fn[T, U, V](f: U -> V, g: T -> U) {
    is_injective_fn(f) and is_injective_fn(g) implies is_injective_fn(compose(f, g))
} by {
    if is_injective_fn(f) and is_injective_fn(g) {
        forall(x1: T, x2: T) {
            if compose(f, g)(x1) = compose(f, g)(x2) {
                compose(f, g, x1) = f(g(x1))
                compose(f, g, x2) = f(g(x2))
                f(g(x1)) = f(g(x2))
                injective_fn_eq(f, g(x1), g(x2))
                g(x1) = g(x2)
                injective_fn_eq(g, x1, x2)
                x1 = x2
            }
        }
    }
}

/// If a composite is injective, then its right factor is injective.
theorem compose_injective_fn_imp_right_factor_injective[T, U, V](f: U -> V, g: T -> U) {
    is_injective_fn(compose(f, g)) implies is_injective_fn(g)
} by {
    if is_injective_fn(compose(f, g)) {
        forall(x1: T, x2: T) {
            if g(x1) = g(x2) {
                compose(f, g, x1) = f(g(x1))
                compose(f, g, x2) = f(g(x2))
                f(g(x1)) = f(g(x2))
                compose(f, g, x1) = compose(f, g, x2)
                injective_fn_eq(compose(f, g), x1, x2)
                x1 = x2
            }
        }
        is_injective_fn(g)
    }
}

/// An injective left factor can be cancelled from an equality of composites.
theorem compose_cancel_left_of_injective_fn[T, U, V](left: U -> V, f: T -> U, g: T -> U) {
    is_injective_fn(left) and compose(left, f) = compose(left, g) implies f = g
} by {
    if is_injective_fn(left) and compose(left, f) = compose(left, g) {
        forall(x: T) {
            function_eq_apply(compose(left, f), compose(left, g), x)
            compose(left, f, x) = compose(left, g, x)
            compose(left, f, x) = left(f(x))
            compose(left, g, x) = left(g(x))
            left(f(x)) = left(g(x))
            injective_fn_eq(left, f(x), g(x))
            f(x) = g(x)
        }
        function_extensionality(f, g)
        f = g
    }
}

/// Surjectivity provides a preimage for every codomain point.
theorem surjective_fn_has_preimage[T, U](f: T -> U, y: U) {
    is_surjective_fn(f) implies exists(x: T) {
        f(x) = y
    }
} by {
    if is_surjective_fn(f) {
        exists(x: T) {
            f(x) = y
        }
    }
}

/// If a composite is surjective, then its left factor is surjective.
theorem compose_surjective_fn_imp_left_factor_surjective[T, U, V](f: U -> V, g: T -> U) {
    is_surjective_fn(compose(f, g)) implies is_surjective_fn(f)
} by {
    if is_surjective_fn(compose(f, g)) {
        forall(z: V) {
            surjective_fn_has_preimage(compose(f, g), z)
            let x: T satisfy {
                compose(f, g)(x) = z
            }
            let y = g(x)
            compose(f, g, x) = f(g(x))
            f(y) = z
            exists(preimage: U) {
                f(preimage) = z
            }
        }
        is_surjective_fn(f)
    }
}

/// A surjective right factor can be cancelled from an equality of composites.
theorem compose_cancel_right_of_surjective_fn[T, U, V](right: T -> U, f: U -> V, g: U -> V) {
    is_surjective_fn(right) and compose(f, right) = compose(g, right) implies f = g
} by {
    if is_surjective_fn(right) and compose(f, right) = compose(g, right) {
        forall(y: U) {
            surjective_fn_has_preimage(right, y)
            let x: T satisfy {
                right(x) = y
            }
            function_eq_apply(compose(f, right), compose(g, right), x)
            compose(f, right, x) = compose(g, right, x)
            compose(f, right, x) = f(right(x))
            compose(g, right, x) = g(right(x))
            f(y) = g(y)
        }
        function_extensionality(f, g)
        f = g
    }
}

/// A left factor with a chosen left inverse can be cancelled from composite equality.
theorem compose_cancel_left_of_left_inverse_fn[T, U, V](left: U -> V, invleft: V -> U, f: T -> U, g: T -> U) {
    is_left_inverse_fn(left, invleft) and compose(left, f) = compose(left, g) implies f = g
} by {
    if is_left_inverse_fn(left, invleft) and compose(left, f) = compose(left, g) {
        left_inverse_fn_imp_injective_fn(left, invleft)
        is_injective_fn(left)
        compose_cancel_left_of_injective_fn(left, f, g)
        f = g
    }
}

/// A right factor with a chosen right inverse can be cancelled from composite equality.
theorem compose_cancel_right_of_right_inverse_fn[T, U, V](right: T -> U, invright: U -> T, f: U -> V, g: U -> V) {
    is_right_inverse_fn(right, invright) and compose(f, right) = compose(g, right) implies f = g
} by {
    if is_right_inverse_fn(right, invright) and compose(f, right) = compose(g, right) {
        right_inverse_fn_imp_surjective_fn(right, invright)
        is_surjective_fn(right)
        compose_cancel_right_of_surjective_fn(right, f, g)
        f = g
    }
}

/// A chosen preimage of a point under a surjective function.
let inverse_fn_at[T: Inhabited, U](f: T -> U, y: U) -> result: T satisfy {
    is_surjective_fn(f) implies f(result) = y
} by {
    if is_surjective_fn(f) {
        surjective_fn_has_preimage(f, y)
        let x: T satisfy {
            f(x) = y
        }
        f(x) = y
    }
}

/// A chosen inverse for a surjective function.
define inverse_fn[T: Inhabited, U](f: T -> U, y: U) -> T {
    inverse_fn_at(f, y)
}

/// A chosen inverse is a right inverse for a surjective function.
theorem inverse_fn_apply_of_surjective[T: Inhabited, U](f: T -> U, y: U) {
    is_surjective_fn(f) implies f(inverse_fn(f, y)) = y
} by {
    if is_surjective_fn(f) {
        inverse_fn(f, y) = inverse_fn_at(f, y)
        f(inverse_fn(f, y)) = y
    }
}

/// A chosen inverse gives back the original point for a bijective function.
theorem inverse_fn_apply_image_of_bijection[T: Inhabited, U](f: T -> U, x: T) {
    is_bijection_fn(f) implies inverse_fn(f, f(x)) = x
} by {
    if is_bijection_fn(f) {
        bijection_fn_is_surjective(f)
        is_surjective_fn(f)
        inverse_fn_apply_of_surjective(f, f(x))
        f(inverse_fn(f, f(x))) = f(x)
        bijection_fn_is_injective(f)
        is_injective_fn(f)
        injective_fn_eq(f, inverse_fn(f, f(x)), x)
        inverse_fn(f, f(x)) = x
    }
}

/// The chosen inverse of a surjective function is a right inverse.
theorem inverse_fn_is_right_inverse_of_surjective[T: Inhabited, U](f: T -> U) {
    is_surjective_fn(f) implies is_right_inverse_fn(f, inverse_fn(f))
} by {
    if is_surjective_fn(f) {
        forall(y: U) {
            inverse_fn_apply_of_surjective(f, y)
            f(inverse_fn(f, y)) = y
        }
        is_right_inverse_fn(f, inverse_fn(f))
    }
}

/// The chosen inverse of a bijection is a left inverse.
theorem inverse_fn_is_left_inverse_of_bijection[T: Inhabited, U](f: T -> U) {
    is_bijection_fn(f) implies is_left_inverse_fn(f, inverse_fn(f))
} by {
    if is_bijection_fn(f) {
        forall(x: T) {
            inverse_fn_apply_image_of_bijection(f, x)
            inverse_fn(f, f(x)) = x
        }
        is_left_inverse_fn(f, inverse_fn(f))
    }
}

/// The chosen inverse of a bijection is a two-sided inverse.
theorem inverse_fn_is_two_sided_inverse_of_bijection[T: Inhabited, U](f: T -> U) {
    is_bijection_fn(f) implies is_two_sided_inverse_fn(f, inverse_fn(f))
} by {
    if is_bijection_fn(f) {
        inverse_fn_is_left_inverse_of_bijection(f)
        is_left_inverse_fn(f, inverse_fn(f))
        bijection_fn_is_surjective(f)
        is_surjective_fn(f)
        inverse_fn_is_right_inverse_of_surjective(f)
        is_right_inverse_fn(f, inverse_fn(f))
        is_two_sided_inverse_fn(f, inverse_fn(f))
    }
}

/// The chosen inverse is the unique two-sided inverse of a bijection.
theorem inverse_fn_eq_of_two_sided_inverse[T: Inhabited, U](f: T -> U, g: U -> T) {
    is_bijection_fn(f) and is_two_sided_inverse_fn(f, g) implies inverse_fn(f) = g
} by {
    if is_bijection_fn(f) and is_two_sided_inverse_fn(f, g) {
        inverse_fn_is_two_sided_inverse_of_bijection(f)
        is_two_sided_inverse_fn(f, inverse_fn(f))
        two_sided_inverse_fn_unique(f, inverse_fn(f), g)
        inverse_fn(f) = g
    }
}

/// The chosen inverse of a bijection is injective.
theorem inverse_fn_is_injective_of_bijection[T: Inhabited, U](f: T -> U) {
    is_bijection_fn(f) implies is_injective_fn(inverse_fn(f))
} by {
    if is_bijection_fn(f) {
        bijection_fn_is_surjective(f)
        is_surjective_fn(f)
        forall(y1: U, y2: U) {
            if inverse_fn(f, y1) = inverse_fn(f, y2) {
                inverse_fn_apply_of_surjective(f, y1)
                inverse_fn_apply_of_surjective(f, y2)
                f(inverse_fn(f, y1)) = y1
                f(inverse_fn(f, y2)) = y2
                y1 = y2
            }
        }
    }
}

/// The chosen inverse of a bijection is surjective.
theorem inverse_fn_is_surjective_of_bijection[T: Inhabited, U](f: T -> U) {
    is_bijection_fn(f) implies is_surjective_fn(inverse_fn(f))
} by {
    if is_bijection_fn(f) {
        forall(x: T) {
            exists(y: U) {
                y = f(x) and inverse_fn(f, y) = x
            }
        }
    }
}

/// The chosen inverse of a bijection is bijective.
theorem inverse_fn_is_bijection_of_bijection[T: Inhabited, U](f: T -> U) {
    is_bijection_fn(f) implies is_bijection_fn(inverse_fn(f))
} by {
    if is_bijection_fn(f) {
        inverse_fn_is_injective_of_bijection(f)
        is_injective_fn(inverse_fn(f))
        inverse_fn_is_surjective_of_bijection(f)
        is_surjective_fn(inverse_fn(f))
        is_bijection_fn(inverse_fn(f))
    }
}

/// Composition preserves surjectivity.
theorem compose_surjective_fn[T, U, V](f: U -> V, g: T -> U) {
    is_surjective_fn(f) and is_surjective_fn(g) implies is_surjective_fn(compose(f, g))
} by {
    if is_surjective_fn(f) and is_surjective_fn(g) {
        forall(z: V) {
            surjective_fn_has_preimage(f, z)
            let y: U satisfy {
                f(y) = z
            }
            surjective_fn_has_preimage(g, y)
            let x: T satisfy {
                g(x) = y
            }
            exists(x0: T) {
                compose(f, g, x0) = z
            }
        }
    }
}

/// Composition preserves bijectivity.
theorem compose_bijection_fn[T, U, V](f: U -> V, g: T -> U) {
    is_bijection_fn(f) and is_bijection_fn(g) implies is_bijection_fn(compose(f, g))
} by {
    if is_bijection_fn(f) and is_bijection_fn(g) {
        bijection_fn_is_injective(f)
        bijection_fn_is_injective(g)
        compose_injective_fn(f, g)
        is_injective_fn(compose(f, g))
        bijection_fn_is_surjective(f)
        bijection_fn_is_surjective(g)
        compose_surjective_fn(f, g)
        is_surjective_fn(compose(f, g))
        is_bijection_fn(compose(f, g))
    }
}

/// If a composite is bijective, then its right factor is injective.
theorem compose_bijection_fn_imp_right_factor_injective[T, U, V](f: U -> V, g: T -> U) {
    is_bijection_fn(compose(f, g)) implies is_injective_fn(g)
} by {
    if is_bijection_fn(compose(f, g)) {
        bijection_fn_is_injective(compose(f, g))
        is_injective_fn(compose(f, g))
        compose_injective_fn_imp_right_factor_injective(f, g)
        is_injective_fn(g)
    }
}

/// If a composite is bijective, then its left factor is surjective.
theorem compose_bijection_fn_imp_left_factor_surjective[T, U, V](f: U -> V, g: T -> U) {
    is_bijection_fn(compose(f, g)) implies is_surjective_fn(f)
} by {
    if is_bijection_fn(compose(f, g)) {
        bijection_fn_is_surjective(compose(f, g))
        is_surjective_fn(compose(f, g))
        compose_surjective_fn_imp_left_factor_surjective(f, g)
        is_surjective_fn(f)
    }
}

/// If a composite is bijective and the left factor is injective, then the right factor is bijective.
theorem compose_bijection_fn_and_left_injective_imp_right_bijection[T, U, V](f: U -> V, g: T -> U) {
    is_bijection_fn(compose(f, g)) and is_injective_fn(f) implies is_bijection_fn(g)
} by {
    if is_bijection_fn(compose(f, g)) and is_injective_fn(f) {
        compose_bijection_fn_imp_right_factor_injective(f, g)
        is_injective_fn(g)

        bijection_fn_is_surjective(compose(f, g))
        is_surjective_fn(compose(f, g))
        forall(y: U) {
            surjective_fn_has_preimage(compose(f, g), f(y))
            let x: T satisfy {
                compose(f, g)(x) = f(y)
            }
            compose(f, g, x) = f(g(x))
            f(g(x)) = f(y)
            injective_fn_eq(f, g(x), y)
            g(x) = y
            exists(preimage: T) {
                g(preimage) = y
            }
        }
        is_surjective_fn(g)
        is_bijection_fn(g)
    }
}

/// If a composite is bijective and the right factor is surjective, then the left factor is bijective.
theorem compose_bijection_fn_and_right_surjective_imp_left_bijection[T, U, V](f: U -> V, g: T -> U) {
    is_bijection_fn(compose(f, g)) and is_surjective_fn(g) implies is_bijection_fn(f)
} by {
    if is_bijection_fn(compose(f, g)) and is_surjective_fn(g) {
        compose_bijection_fn_imp_left_factor_surjective(f, g)
        is_surjective_fn(f)

        bijection_fn_is_injective(compose(f, g))
        is_injective_fn(compose(f, g))
        forall(y1: U, y2: U) {
            if f(y1) = f(y2) {
                surjective_fn_has_preimage(g, y1)
                let x1: T satisfy {
                    g(x1) = y1
                }
                surjective_fn_has_preimage(g, y2)
                let x2: T satisfy {
                    g(x2) = y2
                }
                compose(f, g, x1) = f(g(x1))
                compose(f, g, x2) = f(g(x2))
                compose(f, g, x1) = compose(f, g, x2)
                injective_fn_eq(compose(f, g), x1, x2)
                x1 = x2
                g(x1) = g(x2)
                y1 = y2
            }
        }
        is_injective_fn(f)
        is_bijection_fn(f)
    }
}

/// If a composite is injective and its right factor is surjective, then the left factor is injective.
theorem compose_injective_fn_imp_left_factor_injective_of_right_surjective[T, U, V](f: U -> V, g: T -> U) {
    is_injective_fn(compose(f, g)) and is_surjective_fn(g) implies is_injective_fn(f)
} by {
    if is_injective_fn(compose(f, g)) and is_surjective_fn(g) {
        forall(y1: U, y2: U) {
            if f(y1) = f(y2) {
                surjective_fn_has_preimage(g, y1)
                let x1: T satisfy {
                    g(x1) = y1
                }
                surjective_fn_has_preimage(g, y2)
                let x2: T satisfy {
                    g(x2) = y2
                }
                compose(f, g, x1) = f(g(x1))
                compose(f, g, x2) = f(g(x2))
                compose(f, g, x1) = compose(f, g, x2)
                injective_fn_eq(compose(f, g), x1, x2)
                x1 = x2
                g(x1) = g(x2)
                y1 = y2
            }
        }
        is_injective_fn(f)
    }
}

/// If a composite is surjective and its left factor is injective, then the right factor is surjective.
theorem compose_surjective_fn_imp_right_factor_surjective_of_left_injective[T, U, V](f: U -> V, g: T -> U) {
    is_surjective_fn(compose(f, g)) and is_injective_fn(f) implies is_surjective_fn(g)
} by {
    if is_surjective_fn(compose(f, g)) and is_injective_fn(f) {
        forall(y: U) {
            surjective_fn_has_preimage(compose(f, g), f(y))
            let x: T satisfy {
                compose(f, g)(x) = f(y)
            }
            compose(f, g, x) = f(g(x))
            f(g(x)) = f(y)
            injective_fn_eq(f, g(x), y)
            g(x) = y
            exists(preimage: T) {
                g(preimage) = y
            }
        }
        is_surjective_fn(g)
    }
}

/// An injective composite with a surjective right factor makes that right factor bijective.
theorem compose_injective_fn_and_right_surjective_imp_right_factor_bijection[T, U, V](f: U -> V, g: T -> U) {
    is_injective_fn(compose(f, g)) and is_surjective_fn(g) implies is_bijection_fn(g)
} by {
    if is_injective_fn(compose(f, g)) and is_surjective_fn(g) {
        compose_injective_fn_imp_right_factor_injective(f, g)
        is_injective_fn(g)
        is_bijection_fn(g)
    }
}

/// A surjective composite with an injective left factor makes that left factor bijective.
theorem compose_surjective_fn_and_left_injective_imp_left_factor_bijection[T, U, V](f: U -> V, g: T -> U) {
    is_surjective_fn(compose(f, g)) and is_injective_fn(f) implies is_bijection_fn(f)
} by {
    if is_surjective_fn(compose(f, g)) and is_injective_fn(f) {
        compose_surjective_fn_imp_left_factor_surjective(f, g)
        is_surjective_fn(f)
        is_bijection_fn(f)
    }
}

/// With a surjective right factor, composite injectivity is equivalent to injectivity of both factors.
theorem compose_injective_fn_iff_factors_injective_of_right_surjective[T, U, V](f: U -> V, g: T -> U) {
    is_surjective_fn(g) implies (is_injective_fn(compose(f, g)) = (is_injective_fn(f) and is_injective_fn(g)))
} by {
    if is_surjective_fn(g) {
        if is_injective_fn(compose(f, g)) {
            compose_injective_fn_imp_left_factor_injective_of_right_surjective(f, g)
            is_injective_fn(f)
            compose_injective_fn_imp_right_factor_injective(f, g)
            is_injective_fn(g)
            is_injective_fn(f) and is_injective_fn(g)
        }
        if is_injective_fn(f) and is_injective_fn(g) {
            compose_injective_fn(f, g)
            is_injective_fn(compose(f, g))
        }
        is_injective_fn(compose(f, g)) = (is_injective_fn(f) and is_injective_fn(g))
    }
}

/// With an injective left factor, composite surjectivity is equivalent to surjectivity of both factors.
theorem compose_surjective_fn_iff_factors_surjective_of_left_injective[T, U, V](f: U -> V, g: T -> U) {
    is_injective_fn(f) implies (is_surjective_fn(compose(f, g)) = (is_surjective_fn(f) and is_surjective_fn(g)))
} by {
    if is_injective_fn(f) {
        if is_surjective_fn(compose(f, g)) {
            compose_surjective_fn_imp_left_factor_surjective(f, g)
            is_surjective_fn(f)
            compose_surjective_fn_imp_right_factor_surjective_of_left_injective(f, g)
            is_surjective_fn(g)
            is_surjective_fn(f) and is_surjective_fn(g)
        }
        if is_surjective_fn(f) and is_surjective_fn(g) {
            compose_surjective_fn(f, g)
            is_surjective_fn(compose(f, g))
        }
        is_surjective_fn(compose(f, g)) = (is_surjective_fn(f) and is_surjective_fn(g))
    }
}

/// Composing on the right by a bijection reflects and preserves injectivity of the left factor.
theorem compose_injective_fn_iff_left_factor_injective_of_right_bijection[T, U, V](f: U -> V, g: T -> U) {
    is_bijection_fn(g) implies (is_injective_fn(compose(f, g)) = is_injective_fn(f))
} by {
    if is_bijection_fn(g) {
        bijection_fn_is_surjective(g)
        is_surjective_fn(g)
        bijection_fn_is_injective(g)
        is_injective_fn(g)
        if is_injective_fn(compose(f, g)) {
            compose_injective_fn_imp_left_factor_injective_of_right_surjective(f, g)
            is_injective_fn(f)
        }
        if is_injective_fn(f) {
            compose_injective_fn(f, g)
            is_injective_fn(compose(f, g))
        }
        is_injective_fn(compose(f, g)) = is_injective_fn(f)
    }
}

/// Composing on the left by a bijection reflects and preserves surjectivity of the right factor.
theorem compose_surjective_fn_iff_right_factor_surjective_of_left_bijection[T, U, V](f: U -> V, g: T -> U) {
    is_bijection_fn(f) implies (is_surjective_fn(compose(f, g)) = is_surjective_fn(g))
} by {
    if is_bijection_fn(f) {
        bijection_fn_is_injective(f)
        is_injective_fn(f)
        bijection_fn_is_surjective(f)
        is_surjective_fn(f)
        if is_surjective_fn(compose(f, g)) {
            compose_surjective_fn_imp_right_factor_surjective_of_left_injective(f, g)
            is_surjective_fn(g)
        }
        if is_surjective_fn(g) {
            compose_surjective_fn(f, g)
            is_surjective_fn(compose(f, g))
        }
        is_surjective_fn(compose(f, g)) = is_surjective_fn(g)
    }
}

/// With an injective left factor and surjective right factor, bijectivity reflects to exactly both factors.
theorem compose_bijection_fn_iff_factors_bijection_of_left_injective_right_surjective[T, U, V](f: U -> V, g: T -> U) {
    is_injective_fn(f) and is_surjective_fn(g)
    implies (is_bijection_fn(compose(f, g)) = (is_bijection_fn(f) and is_bijection_fn(g)))
} by {
    if is_injective_fn(f) and is_surjective_fn(g) {
        if is_bijection_fn(compose(f, g)) {
            compose_bijection_fn_and_right_surjective_imp_left_bijection(f, g)
            is_bijection_fn(f)
            compose_bijection_fn_and_left_injective_imp_right_bijection(f, g)
            is_bijection_fn(g)
            is_bijection_fn(f) and is_bijection_fn(g)
        }
        if is_bijection_fn(f) and is_bijection_fn(g) {
            compose_bijection_fn(f, g)
            is_bijection_fn(compose(f, g))
        }
        is_bijection_fn(compose(f, g)) = (is_bijection_fn(f) and is_bijection_fn(g))
    }
}

/// A bijection between two types.
structure Bijection[T, U] {
    /// The underlying map.
    map: T -> U
} constraint {
    is_bijection_fn(map)
}

/// Construction of a bijection remembers the underlying map.
theorem bijection_new_map[T, U](f: T -> U, e: Bijection[T, U]) {
    Bijection[T, U].new(f) = Option.some(e) implies e.map = f
}

/// Every bijection is reconstructed from its underlying map.
theorem bijection_new_self[T, U](e: Bijection[T, U]) {
    Bijection[T, U].new(e.map) = Option.some(e)
}

/// Equality of options containing bijections is equality of bijections.
theorem bijection_some_injective[T, U](e: Bijection[T, U], h: Bijection[T, U]) {
    Option.some(e) = Option.some(h) implies e = h
}

/// Bijections are equal when their underlying maps are equal.
theorem bijection_ext[T, U](e: Bijection[T, U], h: Bijection[T, U]) {
    e.map = h.map implies e = h
} by {
    if e.map = h.map {
        Bijection[T, U].new(e.map) = Option.some(e)
        Bijection[T, U].new(h.map) = Option.some(h)
        Option.some(e) = Option.some(h)
        bijection_some_injective(e, h)
    }
}

/// Equal bijections have equal underlying maps.
theorem bijection_eq_map[T, U](e: Bijection[T, U], h: Bijection[T, U]) {
    e = h implies e.map = h.map
}

/// The underlying map of a bijection is bijective.
theorem bijection_map_is_bijection[T, U](e: Bijection[T, U]) {
    is_bijection_fn(e.map)
}

/// The underlying map of a bijection is injective.
theorem bijection_map_is_injective[T, U](e: Bijection[T, U]) {
    is_injective_fn(e.map)
} by {
    bijection_map_is_bijection(e)
    bijection_fn_is_injective(e.map)
}

/// The underlying map of a bijection is surjective.
theorem bijection_map_is_surjective[T, U](e: Bijection[T, U]) {
    is_surjective_fn(e.map)
} by {
    bijection_map_is_bijection(e)
    bijection_fn_is_surjective(e.map)
}

/// The identity map as a bijection.
let identity_bijection[T](anchor: T) -> result: Bijection[T, T] satisfy {
    result.map = identity_fn[T]
} by {
    is_injective_fn(identity_fn[T])
    is_surjective_fn(identity_fn[T])
    is_bijection_fn(identity_fn[T])
    let e: Bijection[T, T] satisfy {
        Bijection[T, T].new(identity_fn[T]) = Option.some(e)
    }
    bijection_new_map(identity_fn[T], e)
    e.map = identity_fn[T]
}

/// The identity bijection has the identity map.
theorem identity_bijection_map[T](anchor: T) {
    identity_bijection(anchor).map = identity_fn[T]
}

/// The inverse map of a bijection as a bijection.
let inverse_bijection[T: Inhabited, U](e: Bijection[T, U]) -> result: Bijection[U, T] satisfy {
    result.map = inverse_fn(e.map)
} by {
    bijection_map_is_bijection(e)
    inverse_fn_is_bijection_of_bijection(e.map)
    is_bijection_fn(inverse_fn(e.map))
    let h: Bijection[U, T] satisfy {
        Bijection[U, T].new(inverse_fn(e.map)) = Option.some(h)
    }
    bijection_new_map(inverse_fn(e.map), h)
    h.map = inverse_fn(e.map)
}

/// The inverse bijection has the chosen inverse map.
theorem inverse_bijection_map[T: Inhabited, U](e: Bijection[T, U]) {
    inverse_bijection(e).map = inverse_fn(e.map)
}

/// A bijection followed by its inverse is the identity on the source.
theorem inverse_bijection_apply_image[T: Inhabited, U](e: Bijection[T, U], x: T) {
    inverse_bijection(e).map(e.map(x)) = x
} by {
    inverse_bijection_map(e)
    inverse_bijection(e).map = inverse_fn(e.map)
    bijection_map_is_bijection(e)
    inverse_fn_apply_image_of_bijection(e.map, x)
}

/// The inverse of a bijection followed by the bijection is the identity on the target.
theorem bijection_apply_inverse[T: Inhabited, U](e: Bijection[T, U], y: U) {
    e.map(inverse_bijection(e).map(y)) = y
} by {
    inverse_bijection_map(e)
    inverse_bijection(e).map = inverse_fn(e.map)
    bijection_map_is_surjective(e)
    inverse_fn_apply_of_surjective(e.map, y)
}

/// The composite of two bijections.
let compose_bijection[T, U, V](e: Bijection[U, V], h: Bijection[T, U]) -> result: Bijection[T, V] satisfy {
    result.map = compose(e.map, h.map)
} by {
    bijection_map_is_bijection(e)
    bijection_map_is_bijection(h)
    compose_bijection_fn(e.map, h.map)
    is_bijection_fn(compose(e.map, h.map))
    let k: Bijection[T, V] satisfy {
        Bijection[T, V].new(compose(e.map, h.map)) = Option.some(k)
    }
    bijection_new_map(compose(e.map, h.map), k)
    k.map = compose(e.map, h.map)
}

/// The composite bijection has the composite map.
theorem compose_bijection_map[T, U, V](e: Bijection[U, V], h: Bijection[T, U]) {
    compose_bijection(e, h).map = compose(e.map, h.map)
}

/// Identity bijections are left identities for composition.
theorem compose_bijection_identity_left[T, U](anchor: U, e: Bijection[T, U]) {
    compose_bijection(identity_bijection(anchor), e) = e
} by {
    compose_bijection_map(identity_bijection(anchor), e)
    identity_bijection_map(anchor)
    compose_identity_left(e.map)
    compose_bijection(identity_bijection(anchor), e).map = e.map
    bijection_ext(compose_bijection(identity_bijection(anchor), e), e)
}

/// Identity bijections are right identities for composition.
theorem compose_bijection_identity_right[T, U](e: Bijection[T, U], anchor: T) {
    compose_bijection(e, identity_bijection(anchor)) = e
} by {
    compose_bijection_map(e, identity_bijection(anchor))
    identity_bijection_map(anchor)
    compose_identity_right(e.map)
    compose_bijection(e, identity_bijection(anchor)).map = e.map
    bijection_ext(compose_bijection(e, identity_bijection(anchor)), e)
}

/// Composition of bijections is associative.
theorem compose_bijection_assoc[A, B, C, D](f: Bijection[C, D], g: Bijection[B, C], h: Bijection[A, B]) {
    compose_bijection(f, compose_bijection(g, h)) = compose_bijection(compose_bijection(f, g), h)
} by {
    compose_bijection_map(f, compose_bijection(g, h))
    compose_bijection_map(g, h)
    compose_bijection_map(compose_bijection(f, g), h)
    compose_bijection_map(f, g)
    compose_assoc(f.map, g.map, h.map)
    compose_bijection(f, compose_bijection(g, h)).map =
        compose_bijection(compose_bijection(f, g), h).map
    bijection_ext(compose_bijection(f, compose_bijection(g, h)), compose_bijection(compose_bijection(f, g), h))
}

/// The inverse of the inverse of a bijection is the original bijection.
theorem inverse_bijection_inverse[T: Inhabited, U: Inhabited](e: Bijection[T, U]) {
    inverse_bijection(inverse_bijection(e)) = e
} by {
    forall(x: T) {
        inverse_bijection_apply_image(e, x)
        inverse_bijection_apply_image(inverse_bijection(e), e.map(x))
        inverse_bijection(inverse_bijection(e)).map(x) = e.map(x)
    }
    function_extensionality(inverse_bijection(inverse_bijection(e)).map, e.map)
    bijection_ext(inverse_bijection(inverse_bijection(e)), e)
}

/// Composing a bijection on the left with its inverse gives the identity on the source.
theorem compose_bijection_inverse_left[T: Inhabited, U](anchor: T, e: Bijection[T, U]) {
    compose_bijection(inverse_bijection(e), e) = identity_bijection(anchor)
} by {
    forall(x: T) {
        compose_bijection_map(inverse_bijection(e), e)
        inverse_bijection_apply_image(e, x)
        identity_bijection_map(anchor)
        compose_bijection(inverse_bijection(e), e).map(x) = x
        identity_bijection(anchor).map(x) = x
        compose_bijection(inverse_bijection(e), e).map(x) = identity_bijection(anchor).map(x)
    }
    function_extensionality(compose_bijection(inverse_bijection(e), e).map, identity_bijection(anchor).map)
    compose_bijection(inverse_bijection(e), e).map = identity_bijection(anchor).map
    bijection_ext(compose_bijection(inverse_bijection(e), e), identity_bijection(anchor))
}

/// Composing a bijection on the right with its inverse gives the identity on the target.
theorem compose_bijection_inverse_right[T: Inhabited, U](e: Bijection[T, U], anchor: U) {
    compose_bijection(e, inverse_bijection(e)) = identity_bijection(anchor)
} by {
    forall(y: U) {
        compose_bijection_map(e, inverse_bijection(e))
        bijection_apply_inverse(e, y)
        identity_bijection_map(anchor)
        compose_bijection(e, inverse_bijection(e)).map(y) = y
        identity_bijection(anchor).map(y) = y
        compose_bijection(e, inverse_bijection(e)).map(y) = identity_bijection(anchor).map(y)
    }
    function_extensionality(compose_bijection(e, inverse_bijection(e)).map, identity_bijection(anchor).map)
    compose_bijection(e, inverse_bijection(e)).map = identity_bijection(anchor).map
    bijection_ext(compose_bijection(e, inverse_bijection(e)), identity_bijection(anchor))
}

/// The inverse of a composite is the composite of inverses in reverse order.
theorem inverse_bijection_compose[T: Inhabited, U: Inhabited, V](e: Bijection[U, V], h: Bijection[T, U]) {
    inverse_bijection(compose_bijection(e, h)) = compose_bijection(inverse_bijection(h), inverse_bijection(e))
} by {
    forall(z: V) {
        let c = compose_bijection(e, h)
        let r = compose_bijection(inverse_bijection(h), inverse_bijection(e))
        compose_bijection_map(e, h)
        compose_bijection_map(inverse_bijection(h), inverse_bijection(e))
        bijection_apply_inverse(c, z)
        bijection_apply_inverse(e, z)
        bijection_apply_inverse(h, inverse_bijection(e).map(z))
        c.map(inverse_bijection(c).map(z)) = z
        r.map(z) = inverse_bijection(h).map(inverse_bijection(e).map(z))
        c.map(r.map(z)) = z
        bijection_map_is_injective(c)
        injective_fn_eq(c.map, inverse_bijection(c).map(z), r.map(z))
        inverse_bijection(c).map(z) = r.map(z)
        inverse_bijection(compose_bijection(e, h)).map(z) =
            compose_bijection(inverse_bijection(h), inverse_bijection(e)).map(z)
    }
    function_extensionality(inverse_bijection(compose_bijection(e, h)).map,
        compose_bijection(inverse_bijection(h), inverse_bijection(e)).map)
    inverse_bijection(compose_bijection(e, h)).map =
        compose_bijection(inverse_bijection(h), inverse_bijection(e)).map
    bijection_ext(inverse_bijection(compose_bijection(e, h)),
        compose_bijection(inverse_bijection(h), inverse_bijection(e)))
}

/// True if `f` is injective.
define is_injective[T](f: T -> T) -> Bool {
    forall(x: T, y: T) {
        f(x) = f(y) implies x = y
    }
}

/// True if `f` is surjective.
define is_surjective[T](f: T -> T) -> Bool {
    forall(y: T) {
        exists(x: T) {
            f(x) = y
        }
    }
}

/// True if `f` is bijective.
define is_bijection[T](f: T -> T) -> Bool {
    is_injective(f) and is_surjective(f)
}

/// Endomorphic injectivity is the same as general injectivity.
theorem is_injective_eq_is_injective_fn[T](f: T -> T) {
    is_injective(f) = is_injective_fn(f)
} by {
    if is_injective(f) {
        forall(x: T, y: T) {
            if f(x) = f(y) {
                x = y
            }
        }
        is_injective_fn(f)
    }
    if is_injective_fn(f) {
        forall(x: T, y: T) {
            if f(x) = f(y) {
                injective_fn_eq(f, x, y)
                x = y
            }
        }
        is_injective(f)
    }
    is_injective(f) = is_injective_fn(f)
}

/// Endomorphic surjectivity is the same as general surjectivity.
theorem is_surjective_eq_is_surjective_fn[T](f: T -> T) {
    is_surjective(f) = is_surjective_fn(f)
} by {
    if is_surjective(f) {
        forall(y: T) {
            exists(x: T) {
                f(x) = y
            }
        }
        is_surjective_fn(f)
    }
    if is_surjective_fn(f) {
        forall(y: T) {
            surjective_fn_has_preimage(f, y)
            exists(x: T) {
                f(x) = y
            }
        }
        is_surjective(f)
    }
    is_surjective(f) = is_surjective_fn(f)
}

/// Endomorphic bijectivity is the same as general bijectivity.
theorem is_bijection_eq_is_bijection_fn[T](f: T -> T) {
    is_bijection(f) = is_bijection_fn(f)
} by {
    is_injective_eq_is_injective_fn(f)
    is_surjective_eq_is_surjective_fn(f)
    if is_bijection(f) {
        is_injective(f)
        is_injective_fn(f)
        is_surjective(f)
        is_surjective_fn(f)
        is_bijection_fn(f)
    }
    if is_bijection_fn(f) {
        bijection_fn_is_injective(f)
        is_injective_fn(f)
        is_injective(f)
        bijection_fn_is_surjective(f)
        is_surjective_fn(f)
        is_surjective(f)
        is_bijection(f)
    }
    is_bijection(f) = is_bijection_fn(f)
}

/// Equality of endofunctions transports injectivity.
theorem function_eq_transport_injective[T](f: T -> T, g: T -> T) {
    f = g and is_injective(f) implies is_injective(g)
} by {
    if f = g and is_injective(f) {
        function_eq_transport_predicate(is_injective[T], f, g)
        is_injective(g)
    }
}

/// Equality of endofunctions transports injectivity in the reverse direction.
theorem function_eq_transport_injective_rev[T](f: T -> T, g: T -> T) {
    f = g and is_injective(g) implies is_injective(f)
} by {
    if f = g and is_injective(g) {
        function_eq_transport_predicate_rev(is_injective[T], f, g)
        is_injective(f)
    }
}

/// Equality of endofunctions transports surjectivity.
theorem function_eq_transport_surjective[T](f: T -> T, g: T -> T) {
    f = g and is_surjective(f) implies is_surjective(g)
} by {
    if f = g and is_surjective(f) {
        function_eq_transport_predicate(is_surjective[T], f, g)
        is_surjective(g)
    }
}

/// Equality of endofunctions transports surjectivity in the reverse direction.
theorem function_eq_transport_surjective_rev[T](f: T -> T, g: T -> T) {
    f = g and is_surjective(g) implies is_surjective(f)
} by {
    if f = g and is_surjective(g) {
        function_eq_transport_predicate_rev(is_surjective[T], f, g)
        is_surjective(f)
    }
}

/// Equality of endofunctions transports bijectivity.
theorem function_eq_transport_bijection[T](f: T -> T, g: T -> T) {
    f = g and is_bijection(f) implies is_bijection(g)
} by {
    if f = g and is_bijection(f) {
        function_eq_transport_predicate(is_bijection[T], f, g)
        is_bijection(g)
    }
}

/// Equality of endofunctions transports bijectivity in the reverse direction.
theorem function_eq_transport_bijection_rev[T](f: T -> T, g: T -> T) {
    f = g and is_bijection(g) implies is_bijection(f)
} by {
    if f = g and is_bijection(g) {
        function_eq_transport_predicate_rev(is_bijection[T], f, g)
        is_bijection(f)
    }
}
