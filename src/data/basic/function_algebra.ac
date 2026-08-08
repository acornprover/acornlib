/// Pointwise algebraic operations on functions.

from algebra.add import Add
from algebra.add_semigroup import AddSemigroup
from algebra.add_comm_semigroup import AddCommSemigroup
from algebra.add_monoid import AddMonoid
from algebra.add_group import AddGroup
from algebra.mul import Mul
from algebra.semigroup import Semigroup
from algebra.comm_semigroup import CommSemigroup
from algebra.one import One
from algebra.zero import Zero
from algebra.neg import Neg
from algebra.monoid.monoid import Monoid
from algebra.group import Group
from semiring import Semiring
from data.basic.functions import function_extensionality

/// The pointwise sum of two functions.
define pointwise_add[T, A: Add](f: T -> A, g: T -> A, t: T) -> A {
    f(t) + g(t)
}

/// The pointwise product of two functions.
define pointwise_mul[T, A: Mul](f: T -> A, g: T -> A, t: T) -> A {
    f(t) * g(t)
}

/// The constant zero function.
define pointwise_zero[T, A: Zero](t: T) -> A {
    A.0
}

/// The constant one function.
define pointwise_one[T, A: One](t: T) -> A {
    A.1
}

/// The pointwise negation of a function.
define pointwise_neg[T, A: Neg](f: T -> A, t: T) -> A {
    -f(t)
}

/// The pointwise inverse of a function.
define pointwise_inverse[T, A: Group](f: T -> A, t: T) -> A {
    f(t).inverse
}

/// The pointwise sum agrees with the sum of values.
theorem pointwise_add_apply[T, A: Add](f: T -> A, g: T -> A, t: T) {
    pointwise_add(f, g, t) = f(t) + g(t)
}

/// The pointwise product agrees with the product of values.
theorem pointwise_mul_apply[T, A: Mul](f: T -> A, g: T -> A, t: T) {
    pointwise_mul(f, g, t) = f(t) * g(t)
}

/// The pointwise zero has value zero everywhere.
theorem pointwise_zero_apply[T, A: Zero](t: T) {
    pointwise_zero[T, A](t) = A.0
}

/// The pointwise one has value one everywhere.
theorem pointwise_one_apply[T, A: One](t: T) {
    pointwise_one[T, A](t) = A.1
}

/// The pointwise negation agrees with the negation of values.
theorem pointwise_neg_apply[T, A: Neg](f: T -> A, t: T) {
    pointwise_neg(f, t) = -f(t)
}

/// The pointwise inverse agrees with the inverse of values.
theorem pointwise_inverse_apply[T, A: Group](f: T -> A, t: T) {
    pointwise_inverse(f, t) = f(t).inverse
}

/// Pointwise addition is associative.
theorem pointwise_add_assoc[T, A: AddSemigroup](f: T -> A, g: T -> A, h: T -> A) {
    pointwise_add(f, pointwise_add(g, h)) = pointwise_add(pointwise_add(f, g), h)
} by {
    forall(t: T) {
        f(t) + (g(t) + h(t)) = (f(t) + g(t)) + h(t)
        pointwise_add(f, pointwise_add(g, h), t) = pointwise_add(pointwise_add(f, g), h, t)
    }
    function_extensionality(pointwise_add(f, pointwise_add(g, h)), pointwise_add(pointwise_add(f, g), h))
}

/// Pointwise addition is commutative.
theorem pointwise_add_comm[T, A: AddCommSemigroup](f: T -> A, g: T -> A) {
    pointwise_add(f, g) = pointwise_add(g, f)
} by {
    forall(t: T) {
        f(t) + g(t) = g(t) + f(t)
        pointwise_add(f, g, t) = pointwise_add(g, f, t)
    }
    function_extensionality(pointwise_add(f, g), pointwise_add(g, f))
}

/// Adding the pointwise zero on the right changes nothing.
theorem pointwise_add_zero_right[T, A: AddMonoid](f: T -> A) {
    pointwise_add(f, pointwise_zero[T, A]) = f
} by {
    forall(t: T) {
        pointwise_add(f, pointwise_zero[T, A], t) = f(t) + A.0
        pointwise_add(f, pointwise_zero[T, A], t) = f(t)
    }
    function_extensionality(pointwise_add(f, pointwise_zero[T, A]), f)
}

/// Adding the pointwise zero on the left changes nothing.
theorem pointwise_add_zero_left[T, A: AddMonoid](f: T -> A) {
    pointwise_add(pointwise_zero[T, A], f) = f
} by {
    forall(t: T) {
        pointwise_add(pointwise_zero[T, A], f, t) = A.0 + f(t)
        A.0 + f(t) = f(t)
        pointwise_add(pointwise_zero[T, A], f, t) = f(t)
    }
    function_extensionality(pointwise_add(pointwise_zero[T, A], f), f)
}

/// Pointwise negation is a right additive inverse.
theorem pointwise_add_neg_right[T, A: AddGroup](f: T -> A) {
    pointwise_add(f, pointwise_neg(f)) = pointwise_zero[T, A]
} by {
    forall(t: T) {
        pointwise_add(f, pointwise_neg(f), t) = f(t) + -f(t)
        f(t) + -f(t) = A.0
        pointwise_add(f, pointwise_neg(f), t) = pointwise_zero[T, A](t)
    }
    function_extensionality(pointwise_add(f, pointwise_neg(f)), pointwise_zero[T, A])
}

/// Pointwise negation is a left additive inverse.
theorem pointwise_add_neg_left[T, A: AddGroup](f: T -> A) {
    pointwise_add(pointwise_neg(f), f) = pointwise_zero[T, A]
} by {
    forall(t: T) {
        pointwise_add(pointwise_neg(f), f, t) = -f(t) + f(t)
        -f(t) + f(t) = A.0
        pointwise_add(pointwise_neg(f), f, t) = pointwise_zero[T, A](t)
    }
    function_extensionality(pointwise_add(pointwise_neg(f), f), pointwise_zero[T, A])
}

/// Pointwise multiplication is associative.
theorem pointwise_mul_assoc[T, A: Semigroup](f: T -> A, g: T -> A, h: T -> A) {
    pointwise_mul(f, pointwise_mul(g, h)) = pointwise_mul(pointwise_mul(f, g), h)
} by {
    forall(t: T) {
        pointwise_mul(f, pointwise_mul(g, h), t) = f(t) * (g(t) * h(t))
        pointwise_mul(pointwise_mul(f, g), h, t) = (f(t) * g(t)) * h(t)
        f(t) * (g(t) * h(t)) = (f(t) * g(t)) * h(t)
        pointwise_mul(f, pointwise_mul(g, h), t) = pointwise_mul(pointwise_mul(f, g), h, t)
    }
    function_extensionality(pointwise_mul(f, pointwise_mul(g, h)), pointwise_mul(pointwise_mul(f, g), h))
}

/// Pointwise multiplication is commutative.
theorem pointwise_mul_comm[T, A: CommSemigroup](f: T -> A, g: T -> A) {
    pointwise_mul(f, g) = pointwise_mul(g, f)
} by {
    forall(t: T) {
        pointwise_mul(f, g, t) = f(t) * g(t)
        pointwise_mul(g, f, t) = g(t) * f(t)
        f(t) * g(t) = g(t) * f(t)
        pointwise_mul(f, g, t) = pointwise_mul(g, f, t)
    }
    function_extensionality(pointwise_mul(f, g), pointwise_mul(g, f))
}

/// Multiplying by the pointwise one on the right changes nothing.
theorem pointwise_mul_one_right[T, A: Monoid](f: T -> A) {
    pointwise_mul(f, pointwise_one[T, A]) = f
} by {
    forall(t: T) {
        pointwise_mul(f, pointwise_one[T, A], t) = f(t) * A.1
        f(t) * A.1 = f(t)
        pointwise_mul(f, pointwise_one[T, A], t) = f(t)
    }
    function_extensionality(pointwise_mul(f, pointwise_one[T, A]), f)
}

/// Multiplying by the pointwise one on the left changes nothing.
theorem pointwise_mul_one_left[T, A: Monoid](f: T -> A) {
    pointwise_mul(pointwise_one[T, A], f) = f
} by {
    forall(t: T) {
        pointwise_mul(pointwise_one[T, A], f, t) = A.1 * f(t)
        A.1 * f(t) = f(t)
        pointwise_mul(pointwise_one[T, A], f, t) = f(t)
    }
    function_extensionality(pointwise_mul(pointwise_one[T, A], f), f)
}

/// Pointwise inversion is a right inverse.
theorem pointwise_mul_inverse_right[T, A: Group](f: T -> A) {
    pointwise_mul(f, pointwise_inverse(f)) = pointwise_one[T, A]
} by {
    forall(t: T) {
        pointwise_mul(f, pointwise_inverse(f), t) = f(t) * f(t).inverse
        f(t) * f(t).inverse = A.1
        pointwise_mul(f, pointwise_inverse(f), t) = pointwise_one[T, A](t)
    }
    function_extensionality(pointwise_mul(f, pointwise_inverse(f)), pointwise_one[T, A])
}

/// Pointwise inversion is a left inverse.
theorem pointwise_mul_inverse_left[T, A: Group](f: T -> A) {
    pointwise_mul(pointwise_inverse(f), f) = pointwise_one[T, A]
} by {
    forall(t: T) {
        pointwise_mul(pointwise_inverse(f), f, t) = f(t).inverse * f(t)
        f(t).inverse * f(t) = A.1
        pointwise_mul(pointwise_inverse(f), f, t) = pointwise_one[T, A](t)
    }
    function_extensionality(pointwise_mul(pointwise_inverse(f), f), pointwise_one[T, A])
}

/// Pointwise multiplication distributes over pointwise addition on the left.
theorem pointwise_mul_distrib_left[T, A: Semiring](f: T -> A, g: T -> A, h: T -> A) {
    pointwise_mul(f, pointwise_add(g, h)) =
    pointwise_add(pointwise_mul(f, g), pointwise_mul(f, h))
} by {
    forall(t: T) {
        pointwise_mul(f, pointwise_add(g, h), t) = f(t) * (g(t) + h(t))
        pointwise_add(pointwise_mul(f, g), pointwise_mul(f, h), t) = f(t) * g(t) + f(t) * h(t)
        f(t) * (g(t) + h(t)) = f(t) * g(t) + f(t) * h(t)
        pointwise_mul(f, pointwise_add(g, h), t) =
        pointwise_add(pointwise_mul(f, g), pointwise_mul(f, h), t)
    }
    function_extensionality(
        pointwise_mul(f, pointwise_add(g, h)),
        pointwise_add(pointwise_mul(f, g), pointwise_mul(f, h))
    )
}

/// Pointwise multiplication distributes over pointwise addition on the right.
theorem pointwise_mul_distrib_right[T, A: Semiring](f: T -> A, g: T -> A, h: T -> A) {
    pointwise_mul(pointwise_add(f, g), h) =
    pointwise_add(pointwise_mul(f, h), pointwise_mul(g, h))
} by {
    forall(t: T) {
        pointwise_mul(pointwise_add(f, g), h, t) = (f(t) + g(t)) * h(t)
        pointwise_add(pointwise_mul(f, h), pointwise_mul(g, h), t) = f(t) * h(t) + g(t) * h(t)
        (f(t) + g(t)) * h(t) = f(t) * h(t) + g(t) * h(t)
        pointwise_mul(pointwise_add(f, g), h, t) =
        pointwise_add(pointwise_mul(f, h), pointwise_mul(g, h), t)
    }
    function_extensionality(
        pointwise_mul(pointwise_add(f, g), h),
        pointwise_add(pointwise_mul(f, h), pointwise_mul(g, h))
    )
}

/// Multiplying the pointwise zero on the right gives the pointwise zero.
theorem pointwise_mul_zero_right[T, A: Semiring](f: T -> A) {
    pointwise_mul(f, pointwise_zero[T, A]) = pointwise_zero[T, A]
} by {
    forall(t: T) {
        pointwise_mul(f, pointwise_zero[T, A], t) = f(t) * A.0
        f(t) * A.0 = A.0
        pointwise_mul(f, pointwise_zero[T, A], t) = pointwise_zero[T, A](t)
    }
    function_extensionality(pointwise_mul(f, pointwise_zero[T, A]), pointwise_zero[T, A])
}

/// Multiplying the pointwise zero on the left gives the pointwise zero.
theorem pointwise_mul_zero_left[T, A: Semiring](f: T -> A) {
    pointwise_mul(pointwise_zero[T, A], f) = pointwise_zero[T, A]
} by {
    forall(t: T) {
        pointwise_mul(pointwise_zero[T, A], f, t) = A.0 * f(t)
        A.0 * f(t) = A.0
        pointwise_mul(pointwise_zero[T, A], f, t) = pointwise_zero[T, A](t)
    }
    function_extensionality(pointwise_mul(pointwise_zero[T, A], f), pointwise_zero[T, A])
}
