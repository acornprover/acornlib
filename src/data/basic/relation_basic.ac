/// Basic algebra of unbundled relations.

from data.basic.functions import binary_function_extensionality, binary_function_eq_transport_predicate,
    binary_function_eq_transport_predicate_rev

define is_reflexive[T](f: (T, T) -> Bool) -> Bool {
    forall(t: T) {
        f(t, t)
    }
}

define is_symmetric[T](f: (T, T) -> Bool) -> Bool {
    forall(x: T, y: T) {
        f(x, y) implies f(y, x)
    }
}

define is_transitive[T](f: (T, T) -> Bool) -> Bool {
    forall(x: T, y: T, z: T) {
        f(x, y) and f(y, z) implies f(x, z)
    }
}

define is_antisymmetric[T](f: (T, T) -> Bool) -> Bool {
    forall(x: T, y: T) {
        f(x, y) and f(y, x) implies x = y
    }
}

/// True if no element is related to itself.
define is_irreflexive[T](f: (T, T) -> Bool) -> Bool {
    forall(x: T) {
        not f(x, x)
    }
}

/// True if every related pair cannot reverse.
define is_asymmetric[T](f: (T, T) -> Bool) -> Bool {
    forall(x: T, y: T) {
        f(x, y) implies not f(y, x)
    }
}

/// True if every pair of elements is comparable by the relation.
define is_total[T](f: (T, T) -> Bool) -> Bool {
    forall(x: T, y: T) {
        f(x, y) or f(y, x)
    }
}

define is_equivalence[T](f: (T, T) -> Bool) -> Bool {
    is_reflexive(f) and is_symmetric(f) and is_transitive(f)
}

/// True if a relation is symmetric and transitive.
define is_partial_equivalence[T](f: (T, T) -> Bool) -> Bool {
    is_symmetric(f) and is_transitive(f)
}

/// The equality relation on a type.
define eq_relation[T](x: T, y: T) -> Bool {
    x = y
}

/// The composite of relations `r : A -> B` and `s : B -> C`.
define relation_compose[A, B, C](r: (A, B) -> Bool, s: (B, C) -> Bool, x: A, z: C) -> Bool {
    exists(y: B) {
        r(x, y) and s(y, z)
    }
}

/// The pointwise intersection of two relations.
define relation_intersection[A, B](r: (A, B) -> Bool, s: (A, B) -> Bool, x: A, y: B) -> Bool {
    r(x, y) and s(x, y)
}

/// The pointwise union of two relations.
define relation_union[A, B](r: (A, B) -> Bool, s: (A, B) -> Bool, x: A, y: B) -> Bool {
    r(x, y) or s(x, y)
}

/// True if every pair related by `r` is also related by `s`.
define relation_subset[A, B](r: (A, B) -> Bool, s: (A, B) -> Bool) -> Bool {
    forall(x: A, y: B) {
        r(x, y) implies s(x, y)
    }
}

/// A relation inclusion applies to any explicitly related pair.
theorem relation_subset_step[A, B](r: (A, B) -> Bool, s: (A, B) -> Bool, x: A, y: B) {
    relation_subset(r, s) and r(x, y) implies s(x, y)
} by {
    if relation_subset(r, s) and r(x, y) {
        s(x, y)
    }
}

/// Every relation is contained in itself.
theorem relation_subset_refl[A, B](r: (A, B) -> Bool) {
    relation_subset(r, r)
} by {
    forall(x: A, y: B) {
        if r(x, y) {
            r(x, y)
        }
    }
}

/// Relation inclusion is transitive.
theorem relation_subset_trans[A, B](r: (A, B) -> Bool, s: (A, B) -> Bool, t: (A, B) -> Bool) {
    relation_subset(r, s) and relation_subset(s, t) implies relation_subset(r, t)
} by {
    if relation_subset(r, s) and relation_subset(s, t) {
        forall(x: A, y: B) {
            if r(x, y) {
                relation_subset_step(r, s, x, y)
                s(x, y)
                relation_subset_step(s, t, x, y)
                t(x, y)
            }
        }
    }
}

/// Equality of relations is equivalent to mutual inclusion.
theorem relation_eq_iff_subset_both[A, B](r: (A, B) -> Bool, s: (A, B) -> Bool) {
    r = s = (relation_subset(r, s) and relation_subset(s, r))
} by {
    if r = s {
        relation_subset_refl(r)
        relation_subset(r, s)
        relation_subset(s, r)
        relation_subset(r, s) and relation_subset(s, r)
    }
    if relation_subset(r, s) and relation_subset(s, r) {
        forall(x: A, y: B) {
            if r(x, y) {
                relation_subset_step(r, s, x, y)
                s(x, y)
            }
            if s(x, y) {
                relation_subset_step(s, r, x, y)
                r(x, y)
            }
            r(x, y) = s(x, y)
        }
        binary_function_extensionality(r, s)
        r = s
    }
    r = s = (relation_subset(r, s) and relation_subset(s, r))
}

/// The intersection relation is contained in its left factor.
theorem relation_intersection_left_subset[A, B](r: (A, B) -> Bool, s: (A, B) -> Bool) {
    relation_subset(relation_intersection(r, s), r)
} by {
    forall(x: A, y: B) {
        if relation_intersection(r, s, x, y) {
            r(x, y)
        }
    }
}

/// The intersection relation is contained in its right factor.
theorem relation_intersection_right_subset[A, B](r: (A, B) -> Bool, s: (A, B) -> Bool) {
    relation_subset(relation_intersection(r, s), s)
} by {
    forall(x: A, y: B) {
        if relation_intersection(r, s, x, y) {
            s(x, y)
        }
    }
}

/// Any relation contained in both factors is contained in their intersection.
theorem relation_subset_intersection[A, B](t: (A, B) -> Bool, r: (A, B) -> Bool, s: (A, B) -> Bool) {
    relation_subset(t, r) and relation_subset(t, s) implies relation_subset(t, relation_intersection(r, s))
} by {
    if relation_subset(t, r) and relation_subset(t, s) {
        forall(x: A, y: B) {
            if t(x, y) {
                relation_subset_step(t, r, x, y)
                r(x, y)
                relation_subset_step(t, s, x, y)
                s(x, y)
                relation_intersection(r, s, x, y)
            }
        }
    }
}

/// Relation intersection is commutative.
theorem relation_intersection_comm[A, B](r: (A, B) -> Bool, s: (A, B) -> Bool) {
    relation_intersection(r, s) = relation_intersection(s, r)
} by {
    let u = relation_intersection(r, s)
    let v = relation_intersection(s, r)
    forall(x: A, y: B) {
        if u(x, y) {
            r(x, y)
            s(x, y)
            v(x, y)
        }
        if v(x, y) {
            s(x, y)
            r(x, y)
            u(x, y)
        }
        u(x, y) = v(x, y)
    }
    binary_function_extensionality(u, v)
}

/// Relation intersection is associative.
theorem relation_intersection_assoc[A, B](
    r: (A, B) -> Bool,
    s: (A, B) -> Bool,
    t: (A, B) -> Bool
) {
    relation_intersection(r, relation_intersection(s, t)) =
    relation_intersection(relation_intersection(r, s), t)
} by {
    let u = relation_intersection(r, relation_intersection(s, t))
    let v = relation_intersection(relation_intersection(r, s), t)
    forall(x: A, y: B) {
        if u(x, y) {
            r(x, y)
            relation_intersection(s, t, x, y)
            s(x, y)
            t(x, y)
            relation_intersection(r, s, x, y)
            v(x, y)
        }
        if v(x, y) {
            relation_intersection(r, s, x, y)
            r(x, y)
            s(x, y)
            t(x, y)
            relation_intersection(s, t, x, y)
            u(x, y)
        }
        u(x, y) = v(x, y)
    }
    binary_function_extensionality(u, v)
}

/// The intersection of a relation with itself is itself.
theorem relation_intersection_idempotent[A, B](r: (A, B) -> Bool) {
    relation_intersection(r, r) = r
} by {
    let u = relation_intersection(r, r)
    let v = r
    forall(x: A, y: B) {
        if u(x, y) {
            u(x, y) = (r(x, y) and r(x, y))
            v(x, y)
        }
        if v(x, y) {
            u(x, y)
        }
        u(x, y) = v(x, y)
    }
    binary_function_extensionality(u, v)
}

/// Relation intersection is monotone in both arguments.
theorem relation_intersection_monotone[A, B](
    r1: (A, B) -> Bool,
    r2: (A, B) -> Bool,
    s1: (A, B) -> Bool,
    s2: (A, B) -> Bool
) {
    relation_subset(r1, r2) and relation_subset(s1, s2) implies
    relation_subset(relation_intersection(r1, s1), relation_intersection(r2, s2))
} by {
    if relation_subset(r1, r2) and relation_subset(s1, s2) {
        forall(x: A, y: B) {
            if relation_intersection(r1, s1, x, y) {
                r1(x, y)
                relation_subset_step(r1, r2, x, y)
                r2(x, y)
                s1(x, y)
                relation_subset_step(s1, s2, x, y)
                s2(x, y)
                relation_intersection(r2, s2, x, y)
            }
        }
    }
}

/// Intersecting with a larger relation on the right leaves the smaller relation unchanged.
theorem relation_intersection_subset_right_eq[A, B](r: (A, B) -> Bool, s: (A, B) -> Bool) {
    relation_subset(r, s) implies relation_intersection(r, s) = r
} by {
    if relation_subset(r, s) {
        let u = relation_intersection(r, s)
        let v = r
        forall(x: A, y: B) {
            if u(x, y) {
                r(x, y)
                v(x, y)
            }
            if v(x, y) {
                relation_subset_step(r, s, x, y)
                s(x, y)
                u(x, y)
            }
            u(x, y) = v(x, y)
        }
        binary_function_extensionality(u, v)
    }
}

/// Intersecting with a larger relation on the left leaves the smaller relation unchanged.
theorem relation_intersection_subset_left_eq[A, B](r: (A, B) -> Bool, s: (A, B) -> Bool) {
    relation_subset(r, s) implies relation_intersection(s, r) = r
} by {
    if relation_subset(r, s) {
        relation_intersection_comm(s, r)
        relation_intersection_subset_right_eq(r, s)
    }
}

/// The left factor is contained in the union relation.
theorem relation_union_left_subset[A, B](r: (A, B) -> Bool, s: (A, B) -> Bool) {
    relation_subset(r, relation_union(r, s))
} by {
    forall(x: A, y: B) {
        if r(x, y) {
            relation_union(r, s, x, y)
        }
    }
}

/// The right factor is contained in the union relation.
theorem relation_union_right_subset[A, B](r: (A, B) -> Bool, s: (A, B) -> Bool) {
    relation_subset(s, relation_union(r, s))
} by {
    forall(x: A, y: B) {
        if s(x, y) {
            relation_union(r, s, x, y)
        }
    }
}

/// A relation containing both factors contains their union.
theorem relation_union_subset[A, B](r: (A, B) -> Bool, s: (A, B) -> Bool, t: (A, B) -> Bool) {
    relation_subset(r, t) and relation_subset(s, t) implies relation_subset(relation_union(r, s), t)
} by {
    if relation_subset(r, t) and relation_subset(s, t) {
        forall(x: A, y: B) {
            if relation_union(r, s, x, y) {
                relation_union(r, s, x, y) = (r(x, y) or s(x, y))
                if r(x, y) {
                    relation_subset_step(r, t, x, y)
                    t(x, y)
                } else {
                    relation_subset_step(s, t, x, y)
                    t(x, y)
                }
            }
        }
    }
}

/// Relation union is monotone in both arguments.
theorem relation_union_monotone[A, B](
    r1: (A, B) -> Bool,
    r2: (A, B) -> Bool,
    s1: (A, B) -> Bool,
    s2: (A, B) -> Bool
) {
    relation_subset(r1, r2) and relation_subset(s1, s2) implies
    relation_subset(relation_union(r1, s1), relation_union(r2, s2))
} by {
    if relation_subset(r1, r2) and relation_subset(s1, s2) {
        forall(x: A, y: B) {
            if relation_union(r1, s1, x, y) {
                relation_union(r1, s1, x, y) = (r1(x, y) or s1(x, y))
                if r1(x, y) {
                    relation_subset_step(r1, r2, x, y)
                    r2(x, y)
                    relation_union(r2, s2, x, y)
                } else {
                    relation_subset_step(s1, s2, x, y)
                    s2(x, y)
                    relation_union(r2, s2, x, y)
                }
            }
        }
    }
}

/// Relation union is commutative.
theorem relation_union_comm[A, B](r: (A, B) -> Bool, s: (A, B) -> Bool) {
    relation_union(r, s) = relation_union(s, r)
} by {
    let u = relation_union(r, s)
    let v = relation_union(s, r)
    forall(x: A, y: B) {
        if u(x, y) {
            u(x, y) = (r(x, y) or s(x, y))
            v(x, y) = (s(x, y) or r(x, y))
            if r(x, y) {
                v(x, y)
            } else {
                v(x, y)
            }
        }
        if v(x, y) {
            v(x, y) = (s(x, y) or r(x, y))
            u(x, y) = (r(x, y) or s(x, y))
            if s(x, y) {
                u(x, y)
            } else {
                u(x, y)
            }
        }
        u(x, y) = v(x, y)
    }
    binary_function_extensionality(u, v)
}

/// Relation union is associative.
theorem relation_union_assoc[A, B](
    r: (A, B) -> Bool,
    s: (A, B) -> Bool,
    t: (A, B) -> Bool
) {
    relation_union(r, relation_union(s, t)) = relation_union(relation_union(r, s), t)
} by {
    let u = relation_union(r, relation_union(s, t))
    let v = relation_union(relation_union(r, s), t)
    forall(x: A, y: B) {
        if u(x, y) {
            u(x, y) = (r(x, y) or relation_union(s, t, x, y))
            if r(x, y) {
                relation_union(r, s, x, y)
                v(x, y)
            } else {
                relation_union(s, t, x, y) = (s(x, y) or t(x, y))
                if s(x, y) {
                    relation_union(r, s, x, y)
                    v(x, y)
                } else {
                    v(x, y)
                }
            }
        }
        if v(x, y) {
            v(x, y) = (relation_union(r, s, x, y) or t(x, y))
            if relation_union(r, s, x, y) {
                relation_union(r, s, x, y) = (r(x, y) or s(x, y))
                if r(x, y) {
                    u(x, y)
                } else {
                    relation_union(s, t, x, y)
                    u(x, y)
                }
            } else {
                relation_union(s, t, x, y)
                u(x, y)
            }
        }
        u(x, y) = v(x, y)
    }
    binary_function_extensionality(u, v)
}

/// The union of a relation with itself is itself.
theorem relation_union_idempotent[A, B](r: (A, B) -> Bool) {
    relation_union(r, r) = r
} by {
    let u = relation_union(r, r)
    let v = r
    forall(x: A, y: B) {
        if u(x, y) {
            u(x, y) = (r(x, y) or r(x, y))
            v(x, y)
        }
        if v(x, y) {
            u(x, y)
        }
        u(x, y) = v(x, y)
    }
    binary_function_extensionality(u, v)
}

/// Union with a smaller relation on the right leaves the larger relation unchanged.
theorem relation_union_subset_right_eq[A, B](r: (A, B) -> Bool, s: (A, B) -> Bool) {
    relation_subset(s, r) implies relation_union(r, s) = r
} by {
    if relation_subset(s, r) {
        let u = relation_union(r, s)
        let v = r
        forall(x: A, y: B) {
            if u(x, y) {
                u(x, y) = (r(x, y) or s(x, y))
                if r(x, y) {
                    v(x, y)
                } else {
                    relation_subset_step(s, r, x, y)
                    v(x, y)
                }
            }
            if v(x, y) {
                u(x, y)
            }
            u(x, y) = v(x, y)
        }
        binary_function_extensionality(u, v)
    }
}

/// Union with a smaller relation on the left leaves the larger relation unchanged.
theorem relation_union_subset_left_eq[A, B](r: (A, B) -> Bool, s: (A, B) -> Bool) {
    relation_subset(s, r) implies relation_union(s, r) = r
} by {
    if relation_subset(s, r) {
        relation_union_comm(s, r)
        relation_union_subset_right_eq(r, s)
    }
}

/// Intersecting a relation with its union by another relation leaves the relation unchanged.
theorem relation_intersection_union_absorb[A, B](r: (A, B) -> Bool, s: (A, B) -> Bool) {
    relation_intersection(r, relation_union(r, s)) = r
} by {
    relation_union_left_subset(r, s)
    relation_intersection_subset_right_eq(r, relation_union(r, s))
}

/// Unioning a relation with its intersection by another relation leaves the relation unchanged.
theorem relation_union_intersection_absorb[A, B](r: (A, B) -> Bool, s: (A, B) -> Bool) {
    relation_union(r, relation_intersection(r, s)) = r
} by {
    relation_intersection_left_subset(r, s)
    relation_union_subset_right_eq(r, relation_intersection(r, s))
}

/// Intersection distributes over relation union on the right.
theorem relation_intersection_union_distrib[A, B](
    r: (A, B) -> Bool,
    s: (A, B) -> Bool,
    t: (A, B) -> Bool
) {
    relation_intersection(r, relation_union(s, t)) =
    relation_union(relation_intersection(r, s), relation_intersection(r, t))
} by {
    let u = relation_intersection(r, relation_union(s, t))
    let v = relation_union(relation_intersection(r, s), relation_intersection(r, t))
    forall(x: A, y: B) {
        if u(x, y) {
            r(x, y)
            relation_union(s, t, x, y)
            relation_union(s, t, x, y) = (s(x, y) or t(x, y))
            if s(x, y) {
                relation_intersection(r, s, x, y)
                v(x, y)
            } else {
                relation_intersection(r, t, x, y)
                v(x, y)
            }
        }
        if v(x, y) {
            v(x, y) = (relation_intersection(r, s, x, y) or relation_intersection(r, t, x, y))
            if relation_intersection(r, s, x, y) {
                r(x, y)
                s(x, y)
                relation_union(s, t, x, y)
                u(x, y)
            } else {
                relation_intersection(r, t, x, y)
                r(x, y)
                t(x, y)
                relation_union(s, t, x, y)
                u(x, y)
            }
        }
        u(x, y) = v(x, y)
    }
    binary_function_extensionality(u, v)
}

/// Union distributes over relation intersection on the right.
theorem relation_union_intersection_distrib[A, B](
    r: (A, B) -> Bool,
    s: (A, B) -> Bool,
    t: (A, B) -> Bool
) {
    relation_union(r, relation_intersection(s, t)) =
    relation_intersection(relation_union(r, s), relation_union(r, t))
} by {
    let u = relation_union(r, relation_intersection(s, t))
    let v = relation_intersection(relation_union(r, s), relation_union(r, t))
    forall(x: A, y: B) {
        if u(x, y) {
            u(x, y) = (r(x, y) or relation_intersection(s, t, x, y))
            if r(x, y) {
                relation_union(r, s, x, y)
                relation_union(r, t, x, y)
                v(x, y)
            } else {
                relation_intersection(s, t, x, y)
                s(x, y)
                t(x, y)
                relation_union(r, s, x, y)
                relation_union(r, t, x, y)
                v(x, y)
            }
        }
        if v(x, y) {
            relation_union(r, s, x, y)
            relation_union(r, t, x, y)
            relation_union(r, s, x, y) = (r(x, y) or s(x, y))
            if r(x, y) {
                u(x, y)
            } else {
                s(x, y)
                relation_union(r, t, x, y) = (r(x, y) or t(x, y))
                if r(x, y) {
                    u(x, y)
                } else {
                    t(x, y)
                    relation_intersection(s, t, x, y)
                    u(x, y)
                }
            }
        }
        u(x, y) = v(x, y)
    }
    binary_function_extensionality(u, v)
}

/// An equivalence relation is reflexive.
theorem equivalence_is_reflexive[T](f: (T, T) -> Bool) {
    is_equivalence(f) implies is_reflexive(f)
} by {
    if is_equivalence(f) {
        is_equivalence(f) = is_reflexive(f) and is_symmetric(f) and is_transitive(f)
        is_reflexive(f)
    }
}

/// A witness for two consecutive relations yields a witness for the composite relation.
theorem relation_compose_intro[A, B, C](r: (A, B) -> Bool, s: (B, C) -> Bool, x: A, y: B, z: C) {
    r(x, y) and s(y, z) implies relation_compose(r, s, x, z)
} by {
    if r(x, y) and s(y, z) {
        exists(w: B) {
            w = y and r(x, w) and s(w, z)
        }
        relation_compose(r, s, x, z)
    }
}

/// Any instance of a composite relation has a middle witness.
theorem relation_compose_has_witness[A, B, C](r: (A, B) -> Bool, s: (B, C) -> Bool, x: A, z: C) {
    relation_compose(r, s, x, z) implies exists(y: B) {
        r(x, y) and s(y, z)
    }
} by {
    if relation_compose(r, s, x, z) {
        exists(y: B) {
            r(x, y) and s(y, z)
        }
    }
}

/// Relation composition is associative.
theorem relation_compose_assoc[A, B, C, D](
    r: (A, B) -> Bool,
    s: (B, C) -> Bool,
    t: (C, D) -> Bool
) {
    relation_compose(relation_compose(r, s), t) = relation_compose(r, relation_compose(s, t))
} by {
    let u = relation_compose(relation_compose(r, s), t)
    let v = relation_compose(r, relation_compose(s, t))
    forall(x: A, z: D) {
        if u(x, z) {
            relation_compose_has_witness(relation_compose(r, s), t, x, z)
            let c: C satisfy {
                relation_compose(r, s, x, c) and t(c, z)
            }
            relation_compose_has_witness(r, s, x, c)
            let b: B satisfy {
                r(x, b) and s(b, c)
            }
            relation_compose_intro(s, t, b, c, z)
            relation_compose(s, t, b, z)
            relation_compose_intro(r, relation_compose(s, t), x, b, z)
            v(x, z)
        }
        if v(x, z) {
            relation_compose_has_witness(r, relation_compose(s, t), x, z)
            let b: B satisfy {
                r(x, b) and relation_compose(s, t, b, z)
            }
            relation_compose_has_witness(s, t, b, z)
            let c: C satisfy {
                s(b, c) and t(c, z)
            }
            relation_compose_intro(r, s, x, b, c)
            relation_compose(r, s, x, c)
            relation_compose_intro(relation_compose(r, s), t, x, c, z)
            u(x, z)
        }
        u(x, z) = v(x, z)
    }
    binary_function_extensionality(u, v)
}

/// Equality is a left identity for relation composition.
theorem relation_compose_eq_relation_left[A, B](r: (A, B) -> Bool) {
    relation_compose(eq_relation[A], r) = r
} by {
    let u = relation_compose(eq_relation[A], r)
    let v = r
    forall(x: A, y: B) {
        if u(x, y) {
            relation_compose_has_witness(eq_relation[A], r, x, y)
            let x0: A satisfy {
                eq_relation[A](x, x0) and r(x0, y)
            }
            eq_relation[A](x, x0)
            x = x0
            v(x, y)
        }
        if v(x, y) {
            eq_relation[A](x, x)
            relation_compose_intro(eq_relation[A], r, x, x, y)
            u(x, y)
        }
        u(x, y) = v(x, y)
    }
    binary_function_extensionality(u, v)
}

/// Equality is a right identity for relation composition.
theorem relation_compose_eq_relation_right[A, B](r: (A, B) -> Bool) {
    relation_compose(r, eq_relation[B]) = r
} by {
    let u = relation_compose(r, eq_relation[B])
    let v = r
    forall(x: A, y: B) {
        if u(x, y) {
            relation_compose_has_witness(r, eq_relation[B], x, y)
            let y0: B satisfy {
                r(x, y0) and eq_relation[B](y0, y)
            }
            eq_relation[B](y0, y)
            y0 = y
            v(x, y)
        }
        if v(x, y) {
            eq_relation[B](y, y)
            relation_compose_intro(r, eq_relation[B], x, y, y)
            u(x, y)
        }
        u(x, y) = v(x, y)
    }
    binary_function_extensionality(u, v)
}

/// Relation composition is monotone in both arguments.
theorem relation_compose_monotone[A, B, C](
    r1: (A, B) -> Bool,
    r2: (A, B) -> Bool,
    s1: (B, C) -> Bool,
    s2: (B, C) -> Bool
) {
    relation_subset(r1, r2) and relation_subset(s1, s2) implies
    relation_subset(relation_compose(r1, s1), relation_compose(r2, s2))
} by {
    if relation_subset(r1, r2) and relation_subset(s1, s2) {
        relation_subset(r1, r2) = forall(x0: A, y0: B) {
            r1(x0, y0) implies r2(x0, y0)
        }
        relation_subset(s1, s2) = forall(y0: B, z0: C) {
            s1(y0, z0) implies s2(y0, z0)
        }
        forall(x: A, z: C) {
            if relation_compose(r1, s1, x, z) {
                relation_compose_has_witness(r1, s1, x, z)
                let y: B satisfy {
                    r1(x, y) and s1(y, z)
                }
                r2(x, y)
                s2(y, z)
                relation_compose_intro(r2, s2, x, y, z)
                relation_compose(r2, s2, x, z)
            }
        }
    }
}

/// Relation composition distributes over union on the left.
theorem relation_compose_union_left[A, B, C](
    r: (A, B) -> Bool,
    s: (A, B) -> Bool,
    t: (B, C) -> Bool
) {
    relation_compose(relation_union(r, s), t) =
        relation_union(relation_compose(r, t), relation_compose(s, t))
} by {
    let u = relation_compose(relation_union(r, s), t)
    let v = relation_union(relation_compose(r, t), relation_compose(s, t))
    forall(x: A, z: C) {
        if u(x, z) {
            relation_compose_has_witness(relation_union(r, s), t, x, z)
            let y: B satisfy {
                relation_union(r, s, x, y) and t(y, z)
            }
            relation_union(r, s, x, y) = (r(x, y) or s(x, y))
            if r(x, y) {
                relation_compose_intro(r, t, x, y, z)
                relation_compose(r, t, x, z)
                v(x, z)
            }
            if s(x, y) {
                relation_compose_intro(s, t, x, y, z)
                relation_compose(s, t, x, z)
                v(x, z)
            }
            v(x, z)
        }
        if v(x, z) {
            v(x, z) = (relation_compose(r, t, x, z) or relation_compose(s, t, x, z))
            if relation_compose(r, t, x, z) {
                relation_compose_has_witness(r, t, x, z)
                let y: B satisfy {
                    r(x, y) and t(y, z)
                }
                relation_union(r, s, x, y)
                relation_compose_intro(relation_union(r, s), t, x, y, z)
                u(x, z)
            }
            if relation_compose(s, t, x, z) {
                relation_compose_has_witness(s, t, x, z)
                let y: B satisfy {
                    s(x, y) and t(y, z)
                }
                relation_union(r, s, x, y)
                relation_compose_intro(relation_union(r, s), t, x, y, z)
                u(x, z)
            }
            u(x, z)
        }
        u(x, z) = v(x, z)
    }
    binary_function_extensionality(u, v)
}

/// Relation composition distributes over union on the right.
theorem relation_compose_union_right[A, B, C](
    r: (A, B) -> Bool,
    s: (B, C) -> Bool,
    t: (B, C) -> Bool
) {
    relation_compose(r, relation_union(s, t)) =
        relation_union(relation_compose(r, s), relation_compose(r, t))
} by {
    let u = relation_compose(r, relation_union(s, t))
    let v = relation_union(relation_compose(r, s), relation_compose(r, t))
    forall(x: A, z: C) {
        if u(x, z) {
            relation_compose_has_witness(r, relation_union(s, t), x, z)
            let y: B satisfy {
                r(x, y) and relation_union(s, t, y, z)
            }
            relation_union(s, t, y, z) = (s(y, z) or t(y, z))
            if s(y, z) {
                relation_compose_intro(r, s, x, y, z)
                relation_compose(r, s, x, z)
                v(x, z)
            }
            if t(y, z) {
                relation_compose_intro(r, t, x, y, z)
                relation_compose(r, t, x, z)
                v(x, z)
            }
            v(x, z)
        }
        if v(x, z) {
            v(x, z) = (relation_compose(r, s, x, z) or relation_compose(r, t, x, z))
            if relation_compose(r, s, x, z) {
                relation_compose_has_witness(r, s, x, z)
                let y: B satisfy {
                    r(x, y) and s(y, z)
                }
                relation_union(s, t, y, z)
                relation_compose_intro(r, relation_union(s, t), x, y, z)
                u(x, z)
            }
            if relation_compose(r, t, x, z) {
                relation_compose_has_witness(r, t, x, z)
                let y: B satisfy {
                    r(x, y) and t(y, z)
                }
                relation_union(s, t, y, z)
                relation_compose_intro(r, relation_union(s, t), x, y, z)
                u(x, z)
            }
            u(x, z)
        }
        u(x, z) = v(x, z)
    }
    binary_function_extensionality(u, v)
}

/// Composition with an intersection on the left lies in the intersection of the compositions.
theorem relation_compose_intersection_left_subset[A, B, C](
    r: (A, B) -> Bool,
    s: (A, B) -> Bool,
    t: (B, C) -> Bool
) {
    relation_subset(
        relation_compose(relation_intersection(r, s), t),
        relation_intersection(relation_compose(r, t), relation_compose(s, t))
    )
} by {
    forall(x: A, z: C) {
        if relation_compose(relation_intersection(r, s), t, x, z) {
            relation_compose_has_witness(relation_intersection(r, s), t, x, z)
            let y: B satisfy {
                relation_intersection(r, s, x, y) and t(y, z)
            }
            relation_intersection(r, s, x, y) = (r(x, y) and s(x, y))
            relation_compose_intro(r, t, x, y, z)
            relation_compose(r, t, x, z)
            relation_compose_intro(s, t, x, y, z)
            relation_compose(s, t, x, z)
            relation_intersection(relation_compose(r, t), relation_compose(s, t), x, z)
        }
    }
}

/// Composition with an intersection on the right lies in the intersection of the compositions.
theorem relation_compose_intersection_right_subset[A, B, C](
    r: (A, B) -> Bool,
    s: (B, C) -> Bool,
    t: (B, C) -> Bool
) {
    relation_subset(
        relation_compose(r, relation_intersection(s, t)),
        relation_intersection(relation_compose(r, s), relation_compose(r, t))
    )
} by {
    forall(x: A, z: C) {
        if relation_compose(r, relation_intersection(s, t), x, z) {
            relation_compose_has_witness(r, relation_intersection(s, t), x, z)
            let y: B satisfy {
                r(x, y) and relation_intersection(s, t, y, z)
            }
            relation_intersection(s, t, y, z) = (s(y, z) and t(y, z))
            relation_compose_intro(r, s, x, y, z)
            relation_compose(r, s, x, z)
            relation_compose_intro(r, t, x, y, z)
            relation_compose(r, t, x, z)
            relation_intersection(relation_compose(r, s), relation_compose(r, t), x, z)
        }
    }
}

/// The converse of a relation swaps its arguments.
define relation_converse[A, B](r: (A, B) -> Bool, y: B, x: A) -> Bool {
    r(x, y)
}

/// Taking converse twice recovers the original relation.
theorem relation_converse_involutive[A, B](r: (A, B) -> Bool) {
    relation_converse(relation_converse(r)) = r
} by {
    let u = relation_converse(relation_converse(r))
    let v = r
    forall(x: A, y: B) {
        u(x, y) = relation_converse(r, y, x)
        relation_converse(r, y, x) = r(x, y)
        u(x, y) = v(x, y)
    }
    binary_function_extensionality(u, v)
}

/// Equality is fixed by converse.
theorem relation_converse_eq_relation[T] {
    relation_converse(eq_relation[T]) = eq_relation[T]
} by {
    let u = relation_converse(eq_relation[T])
    let v = eq_relation[T]
    forall(x: T, y: T) {
        u(x, y) = eq_relation[T](y, x)
        eq_relation[T](y, x) = (y = x)
        y = x = (x = y)
        eq_relation[T](x, y) = (x = y)
        u(x, y) = v(x, y)
    }
    binary_function_extensionality(u, v)
}

/// Converse reverses the order of relation composition.
theorem relation_converse_compose[A, B, C](r: (A, B) -> Bool, s: (B, C) -> Bool) {
    relation_converse(relation_compose(r, s)) = relation_compose(relation_converse(s), relation_converse(r))
} by {
    let u = relation_converse(relation_compose(r, s))
    let v = relation_compose(relation_converse(s), relation_converse(r))
    forall(z: C, x: A) {
        if u(z, x) {
            relation_compose(r, s, x, z)
            relation_compose_has_witness(r, s, x, z)
            let y: B satisfy {
                r(x, y) and s(y, z)
            }
            relation_converse(s, z, y)
            relation_converse(r, y, x)
            relation_compose_intro(relation_converse(s), relation_converse(r), z, y, x)
            v(z, x)
        }
        if v(z, x) {
            relation_compose_has_witness(relation_converse(s), relation_converse(r), z, x)
            let y: B satisfy {
                relation_converse(s, z, y) and relation_converse(r, y, x)
            }
            s(y, z)
            r(x, y)
            relation_compose_intro(r, s, x, y, z)
            relation_compose(r, s, x, z)
            u(z, x)
        }
        u(z, x) = v(z, x)
    }
    binary_function_extensionality(u, v)
}

/// Taking converse is monotone with respect to relation inclusion.
theorem relation_converse_monotone[A, B](r1: (A, B) -> Bool, r2: (A, B) -> Bool) {
    relation_subset(r1, r2) implies relation_subset(relation_converse(r1), relation_converse(r2))
} by {
    if relation_subset(r1, r2) {
        relation_subset(r1, r2) = forall(x: A, y: B) {
            r1(x, y) implies r2(x, y)
        }
        forall(y: B, x: A) {
            if relation_converse(r1, y, x) {
                r1(x, y)
                r2(x, y)
                relation_converse(r2, y, x)
            }
        }
    }
}

/// An asymmetric relation is irreflexive.
theorem asymmetric_imp_irreflexive[T](r: (T, T) -> Bool) {
    is_asymmetric(r) implies is_irreflexive(r)
} by {
    if is_asymmetric(r) {
        is_asymmetric(r) = forall(x0: T, y0: T) {
            r(x0, y0) implies not r(y0, x0)
        }
        forall(x: T) {
            if r(x, x) {
                not r(x, x)
                false
            }
        }
    }
}

/// An asymmetric relation is antisymmetric.
theorem asymmetric_imp_antisymmetric[T](r: (T, T) -> Bool) {
    is_asymmetric(r) implies is_antisymmetric(r)
} by {
    if is_asymmetric(r) {
        is_asymmetric(r) = forall(x0: T, y0: T) {
            r(x0, y0) implies not r(y0, x0)
        }
        forall(x: T, y: T) {
            if r(x, y) and r(y, x) {
                not r(y, x)
                false
            }
        }
    }
}

/// Converse preserves irreflexivity.
theorem relation_converse_is_irreflexive[T](r: (T, T) -> Bool) {
    is_irreflexive(r) implies is_irreflexive(relation_converse(r))
} by {
    if is_irreflexive(r) {
        forall(x: T) {
            if relation_converse(r, x, x) {
                r(x, x)
                false
            }
        }
    }
}

/// Converse preserves reflexivity.
theorem relation_converse_is_reflexive[T](r: (T, T) -> Bool) {
    is_reflexive(r) implies is_reflexive(relation_converse(r))
} by {
    if is_reflexive(r) {
        forall(x: T) {
            is_reflexive(r) = forall(x0: T) {
                r(x0, x0)
            }
            r(x, x)
            relation_converse(r, x, x)
        }
    }
}

/// Converse preserves symmetry.
theorem relation_converse_is_symmetric[T](r: (T, T) -> Bool) {
    is_symmetric(r) implies is_symmetric(relation_converse(r))
} by {
    if is_symmetric(r) {
        forall(x: T, y: T) {
            if relation_converse(r, x, y) {
                relation_converse(r, x, y) = r(y, x)
                r(y, x)
                is_symmetric(r) = forall(x0: T, y0: T) {
                    r(x0, y0) implies r(y0, x0)
                }
                r(x, y)
                relation_converse(r, y, x)
            }
        }
    }
}

/// Converse preserves transitivity.
theorem relation_converse_is_transitive[T](r: (T, T) -> Bool) {
    is_transitive(r) implies is_transitive(relation_converse(r))
} by {
    if is_transitive(r) {
        forall(x: T, y: T, z: T) {
            if relation_converse(r, x, y) and relation_converse(r, y, z) {
                relation_converse(r, x, y) = r(y, x)
                r(y, x)
                relation_converse(r, y, z) = r(z, y)
                r(z, y)
                is_transitive(r) = forall(x0: T, y0: T, z0: T) {
                    r(x0, y0) and r(y0, z0) implies r(x0, z0)
                }
                r(z, x)
                relation_converse(r, x, z)
            }
        }
    }
}

/// Converse preserves asymmetry.
theorem relation_converse_is_asymmetric[T](r: (T, T) -> Bool) {
    is_asymmetric(r) implies is_asymmetric(relation_converse(r))
} by {
    if is_asymmetric(r) {
        is_asymmetric(r) = forall(x0: T, y0: T) {
            r(x0, y0) implies not r(y0, x0)
        }
        forall(x: T, y: T) {
            if relation_converse(r, x, y) {
                relation_converse(r, x, y) = r(y, x)
                r(y, x)
                not r(x, y)
                if relation_converse(r, y, x) {
                    relation_converse(r, y, x) = r(x, y)
                    r(x, y)
                    false
                }
            }
        }
    }
}

/// Converse preserves totality.
theorem relation_converse_is_total[T](r: (T, T) -> Bool) {
    is_total(r) implies is_total(relation_converse(r))
} by {
    if is_total(r) {
        is_total(r) = forall(x0: T, y0: T) {
            r(x0, y0) or r(y0, x0)
        }
        forall(x: T, y: T) {
            r(y, x) or r(x, y)
            if r(y, x) {
                relation_converse(r, x, y)
                relation_converse(r, x, y) or relation_converse(r, y, x)
            } else {
                relation_converse(r, y, x)
                relation_converse(r, x, y) or relation_converse(r, y, x)
            }
        }
    }
}

/// Converse preserves antisymmetry.
theorem relation_converse_is_antisymmetric[T](r: (T, T) -> Bool) {
    is_antisymmetric(r) implies is_antisymmetric(relation_converse(r))
} by {
    if is_antisymmetric(r) {
        is_antisymmetric(r) = forall(x0: T, y0: T) {
            r(x0, y0) and r(y0, x0) implies x0 = y0
        }
        forall(x: T, y: T) {
            if relation_converse(r, x, y) and relation_converse(r, y, x) {
                relation_converse(r, x, y) = r(y, x)
                relation_converse(r, y, x) = r(x, y)
                r(y, x) and r(x, y)
                y = x
                x = y
            }
        }
    }
}

/// Converse commutes with relation intersection.
theorem relation_converse_intersection[A, B](r: (A, B) -> Bool, s: (A, B) -> Bool) {
    relation_converse(relation_intersection(r, s)) =
    relation_intersection(relation_converse(r), relation_converse(s))
} by {
    let u = relation_converse(relation_intersection(r, s))
    let v = relation_intersection(relation_converse(r), relation_converse(s))
    forall(x: B, y: A) {
        if u(x, y) {
            relation_converse(relation_intersection(r, s), x, y)
            relation_intersection(r, s, y, x)
            r(y, x)
            s(y, x)
            relation_converse(r, x, y)
            relation_converse(s, x, y)
            v(x, y)
        }
        if v(x, y) {
            relation_converse(r, x, y)
            relation_converse(s, x, y)
            r(y, x)
            s(y, x)
            relation_intersection(r, s, y, x)
            u(x, y)
        }
        u(x, y) = v(x, y)
    }
    binary_function_extensionality(u, v)
}

/// Converse commutes with relation union.
theorem relation_converse_union[A, B](r: (A, B) -> Bool, s: (A, B) -> Bool) {
    relation_converse(relation_union(r, s)) =
    relation_union(relation_converse(r), relation_converse(s))
} by {
    let u = relation_converse(relation_union(r, s))
    let v = relation_union(relation_converse(r), relation_converse(s))
    forall(x: B, y: A) {
        if u(x, y) {
            relation_converse(relation_union(r, s), x, y)
            relation_union(r, s, y, x)
            relation_union(r, s, y, x) = (r(y, x) or s(y, x))
            if r(y, x) {
                relation_converse(r, x, y)
                v(x, y)
            } else {
                relation_converse(s, x, y)
                v(x, y)
            }
        }
        if v(x, y) {
            v(x, y) = (relation_converse(r, x, y) or relation_converse(s, x, y))
            if relation_converse(r, x, y) {
                r(y, x)
                relation_union(r, s, y, x)
                u(x, y)
            } else {
                s(y, x)
                relation_union(r, s, y, x)
                u(x, y)
            }
        }
        u(x, y) = v(x, y)
    }
    binary_function_extensionality(u, v)
}

/// A symmetric relation contains its converse.
theorem relation_converse_subset_of_symmetric[T](r: (T, T) -> Bool) {
    is_symmetric(r) implies relation_subset(relation_converse(r), r)
} by {
    if is_symmetric(r) {
        is_symmetric(r) = forall(x: T, y: T) {
            r(x, y) implies r(y, x)
        }
        forall(x: T, y: T) {
            if relation_converse(r, x, y) {
                r(y, x)
                r(x, y)
            }
        }
    }
}

/// A symmetric relation is contained in its converse.
theorem relation_subset_converse_of_symmetric[T](r: (T, T) -> Bool) {
    is_symmetric(r) implies relation_subset(r, relation_converse(r))
} by {
    if is_symmetric(r) {
        is_symmetric(r) = forall(x: T, y: T) {
            r(x, y) implies r(y, x)
        }
        forall(x: T, y: T) {
            if r(x, y) {
                r(y, x)
                relation_converse(r, x, y)
            }
        }
    }
}

/// Symmetry identifies each converse value with the original relation value.
theorem symmetric_relation_converse_eq_at[T](r: (T, T) -> Bool, x: T, y: T) {
    is_symmetric(r) implies relation_converse(r, x, y) = r(x, y)
} by {
    if is_symmetric(r) {
        relation_converse_subset_of_symmetric(r)
        relation_subset_converse_of_symmetric(r)
        if relation_converse(r, x, y) {
            relation_subset_step(relation_converse(r), r, x, y)
            r(x, y)
        }
        if r(x, y) {
            relation_subset_step(r, relation_converse(r), x, y)
            relation_converse(r, x, y)
        }
        relation_converse(r, x, y) = r(x, y)
    }
}

/// A symmetric relation is equal to its converse.
theorem symmetric_imp_relation_converse_eq[T](r: (T, T) -> Bool) {
    is_symmetric(r) implies relation_converse(r) = r
} by {
    let u = relation_converse(r)
    let v = r
    if is_symmetric(r) {
        forall(x: T, y: T) {
            symmetric_relation_converse_eq_at(r, x, y)
            u(x, y) = v(x, y)
        }
        binary_function_extensionality(u, v)
    }
}

/// Equality with the converse implies symmetry.
theorem relation_converse_eq_imp_symmetric[T](r: (T, T) -> Bool) {
    relation_converse(r) = r implies is_symmetric(r)
} by {
    if relation_converse(r) = r {
        is_symmetric(r) = forall(x: T, y: T) {
            r(x, y) implies r(y, x)
        }
        forall(x: T, y: T) {
            if r(x, y) {
                relation_converse(r, y, x)
                r(y, x)
            }
        }
    }
}

/// The reflexive closure of a homogeneous relation.
define relation_reflexive_closure[T](r: (T, T) -> Bool, x: T, y: T) -> Bool {
    eq_relation[T](x, y) or r(x, y)
}

/// Every relation is contained in its reflexive closure.
theorem relation_subset_reflexive_closure[T](r: (T, T) -> Bool) {
    relation_subset(r, relation_reflexive_closure(r))
} by {
    forall(x: T, y: T) {
        if r(x, y) {
            relation_reflexive_closure(r, x, y)
        }
    }
}

/// Reflexive closure is reflexive.
theorem relation_reflexive_closure_is_reflexive[T](r: (T, T) -> Bool) {
    is_reflexive(relation_reflexive_closure(r))
} by {
    forall(x: T) {
        eq_relation[T](x, x)
        relation_reflexive_closure(r, x, x)
    }
}

/// Reflexive closure is the smallest reflexive relation containing the original relation.
theorem relation_reflexive_closure_subset_of_reflexive[T](r: (T, T) -> Bool, s: (T, T) -> Bool) {
    is_reflexive(s) and relation_subset(r, s) implies relation_subset(relation_reflexive_closure(r), s)
} by {
    if is_reflexive(s) and relation_subset(r, s) {
        is_reflexive(s) = forall(t: T) {
            s(t, t)
        }
        forall(x: T, y: T) {
            if relation_reflexive_closure(r, x, y) {
                relation_reflexive_closure(r, x, y) = (eq_relation[T](x, y) or r(x, y))
                if eq_relation[T](x, y) {
                    x = y
                    s(x, x)
                    s(x, y)
                } else {
                    relation_subset_step(r, s, x, y)
                    s(x, y)
                }
            }
        }
    }
}

/// The symmetric closure of a homogeneous relation.
define relation_symmetric_closure[T](r: (T, T) -> Bool, x: T, y: T) -> Bool {
    r(x, y) or relation_converse(r, x, y)
}

/// Every relation is contained in its symmetric closure.
theorem relation_subset_symmetric_closure[T](r: (T, T) -> Bool) {
    relation_subset(r, relation_symmetric_closure(r))
} by {
    forall(x: T, y: T) {
        if r(x, y) {
            relation_symmetric_closure(r, x, y)
        }
    }
}

/// Symmetric closure is symmetric.
theorem relation_symmetric_closure_is_symmetric[T](r: (T, T) -> Bool) {
    is_symmetric(relation_symmetric_closure(r))
} by {
    forall(x: T, y: T) {
        if relation_symmetric_closure(r, x, y) {
            relation_symmetric_closure(r, x, y) = (r(x, y) or relation_converse(r, x, y))
            if r(x, y) {
                relation_converse(r, y, x)
                relation_symmetric_closure(r, y, x)
            } else {
                r(y, x)
                relation_symmetric_closure(r, y, x)
            }
        }
    }
}

/// Symmetric closure is the smallest symmetric relation containing the original relation.
theorem relation_symmetric_closure_subset_of_symmetric[T](r: (T, T) -> Bool, s: (T, T) -> Bool) {
    is_symmetric(s) and relation_subset(r, s) implies relation_subset(relation_symmetric_closure(r), s)
} by {
    if is_symmetric(s) and relation_subset(r, s) {
        is_symmetric(s) = forall(x0: T, y0: T) {
            s(x0, y0) implies s(y0, x0)
        }
        forall(x: T, y: T) {
            if relation_symmetric_closure(r, x, y) {
                relation_symmetric_closure(r, x, y) = (r(x, y) or relation_converse(r, x, y))
                if r(x, y) {
                    relation_subset_step(r, s, x, y)
                    s(x, y)
                } else {
                    r(y, x)
                    relation_subset_step(r, s, y, x)
                    s(y, x)
                    s(x, y)
                }
            }
        }
    }
}

/// Reflexive closure is monotone in its argument.
theorem relation_reflexive_closure_monotone[T](r: (T, T) -> Bool, s: (T, T) -> Bool) {
    relation_subset(r, s) implies
        relation_subset(relation_reflexive_closure(r), relation_reflexive_closure(s))
} by {
    if relation_subset(r, s) {
        forall(x: T, y: T) {
            if relation_reflexive_closure(r, x, y) {
                relation_reflexive_closure(r, x, y) = (eq_relation[T](x, y) or r(x, y))
                if eq_relation[T](x, y) {
                    relation_reflexive_closure(s, x, y)
                } else {
                    relation_subset_step(r, s, x, y)
                    s(x, y)
                    relation_reflexive_closure(s, x, y)
                }
            }
        }
    }
}

/// Reflexive closure of a reflexive relation equals itself.
theorem relation_reflexive_closure_eq_self_of_reflexive[T](r: (T, T) -> Bool) {
    is_reflexive(r) implies relation_reflexive_closure(r) = r
} by {
    if is_reflexive(r) {
        relation_reflexive_closure_subset_of_reflexive(r, r)
        relation_subset_refl(r)
        relation_subset(r, r)
        relation_subset(relation_reflexive_closure(r), r)
        relation_subset_reflexive_closure(r)
        relation_subset(r, relation_reflexive_closure(r))
        relation_eq_iff_subset_both(relation_reflexive_closure(r), r)
        relation_reflexive_closure(r) = r
    }
}

/// Reflexive closure is idempotent.
theorem relation_reflexive_closure_idempotent[T](r: (T, T) -> Bool) {
    relation_reflexive_closure(relation_reflexive_closure(r)) = relation_reflexive_closure(r)
} by {
    relation_reflexive_closure_is_reflexive(r)
    relation_reflexive_closure_eq_self_of_reflexive(relation_reflexive_closure(r))
}

/// Symmetric closure is monotone in its argument.
theorem relation_symmetric_closure_monotone[T](r: (T, T) -> Bool, s: (T, T) -> Bool) {
    relation_subset(r, s) implies
        relation_subset(relation_symmetric_closure(r), relation_symmetric_closure(s))
} by {
    if relation_subset(r, s) {
        forall(x: T, y: T) {
            if relation_symmetric_closure(r, x, y) {
                relation_symmetric_closure(r, x, y) = (r(x, y) or relation_converse(r, x, y))
                if r(x, y) {
                    relation_subset_step(r, s, x, y)
                    s(x, y)
                    relation_symmetric_closure(s, x, y)
                } else {
                    r(y, x)
                    relation_subset_step(r, s, y, x)
                    s(y, x)
                    relation_converse(s, x, y)
                    relation_symmetric_closure(s, x, y)
                }
            }
        }
    }
}

/// Symmetric closure of a symmetric relation equals itself.
theorem relation_symmetric_closure_eq_self_of_symmetric[T](r: (T, T) -> Bool) {
    is_symmetric(r) implies relation_symmetric_closure(r) = r
} by {
    if is_symmetric(r) {
        relation_symmetric_closure_subset_of_symmetric(r, r)
        relation_subset_refl(r)
        relation_subset(r, r)
        relation_subset(relation_symmetric_closure(r), r)
        relation_subset_symmetric_closure(r)
        relation_subset(r, relation_symmetric_closure(r))
        relation_eq_iff_subset_both(relation_symmetric_closure(r), r)
        relation_symmetric_closure(r) = r
    }
}

/// Symmetric closure is idempotent.
theorem relation_symmetric_closure_idempotent[T](r: (T, T) -> Bool) {
    relation_symmetric_closure(relation_symmetric_closure(r)) = relation_symmetric_closure(r)
} by {
    relation_symmetric_closure_is_symmetric(r)
    relation_symmetric_closure_eq_self_of_symmetric(relation_symmetric_closure(r))
}

/// An equivalence relation is symmetric.
theorem equivalence_is_symmetric[T](f: (T, T) -> Bool) {
    is_equivalence(f) implies is_symmetric(f)
} by {
    if is_equivalence(f) {
        is_equivalence(f) = is_reflexive(f) and is_symmetric(f) and is_transitive(f)
        is_symmetric(f)
    }
}

/// A partial equivalence relation is symmetric.
theorem partial_equivalence_is_symmetric[T](f: (T, T) -> Bool) {
    is_partial_equivalence(f) implies is_symmetric(f)
} by {
    if is_partial_equivalence(f) {
        is_partial_equivalence(f) = is_symmetric(f) and is_transitive(f)
        is_symmetric(f)
    }
}

/// An equivalence relation is fixed by converse.
theorem equivalence_imp_relation_converse_eq[T](r: (T, T) -> Bool) {
    is_equivalence(r) implies relation_converse(r) = r
} by {
    if is_equivalence(r) {
        equivalence_is_symmetric(r)
        symmetric_imp_relation_converse_eq(r)
    }
}

/// A partial equivalence relation is fixed by converse.
theorem partial_equivalence_imp_relation_converse_eq[T](r: (T, T) -> Bool) {
    is_partial_equivalence(r) implies relation_converse(r) = r
} by {
    if is_partial_equivalence(r) {
        partial_equivalence_is_symmetric(r)
        symmetric_imp_relation_converse_eq(r)
    }
}

/// An equivalence relation is transitive.
theorem equivalence_is_transitive[T](f: (T, T) -> Bool) {
    is_equivalence(f) implies is_transitive(f)
} by {
    if is_equivalence(f) {
        is_equivalence(f) = is_reflexive(f) and is_symmetric(f) and is_transitive(f)
        is_transitive(f)
    }
}

/// A partial equivalence relation is transitive.
theorem partial_equivalence_is_transitive[T](f: (T, T) -> Bool) {
    is_partial_equivalence(f) implies is_transitive(f)
} by {
    if is_partial_equivalence(f) {
        is_partial_equivalence(f) = is_symmetric(f) and is_transitive(f)
        is_transitive(f)
    }
}

/// A reflexive relation holds on each diagonal element.
theorem reflexive_self[T](f: (T, T) -> Bool, x: T) {
    is_reflexive(f) implies f(x, x)
} by {
    if is_reflexive(f) {
        is_reflexive(f) = forall(x0: T) {
            f(x0, x0)
        }
        f(x, x)
    }
}

/// A symmetric relation reverses any related pair.
theorem symmetric_flip[T](f: (T, T) -> Bool, x: T, y: T) {
    is_symmetric(f) and f(x, y) implies f(y, x)
} by {
    if is_symmetric(f) and f(x, y) {
        is_symmetric(f) = forall(x0: T, y0: T) {
            f(x0, y0) implies f(y0, x0)
        }
        f(y, x)
    }
}

/// A partial equivalence relation can reverse any related pair.
theorem partial_equivalence_flip[T](f: (T, T) -> Bool, x: T, y: T) {
    is_partial_equivalence(f) and f(x, y) implies f(y, x)
} by {
    if is_partial_equivalence(f) and f(x, y) {
        partial_equivalence_is_symmetric(f)
        is_symmetric(f)
        symmetric_flip(f, x, y)
        f(y, x)
    }
}

/// A transitive relation composes two consecutive steps.
theorem transitive_step[T](f: (T, T) -> Bool, x: T, y: T, z: T) {
    is_transitive(f) and f(x, y) and f(y, z) implies f(x, z)
} by {
    if is_transitive(f) {
        if f(x, y) and f(y, z) {
            is_transitive(f) = forall(x0: T, y0: T, z0: T) {
                f(x0, y0) and f(y0, z0) implies f(x0, z0)
            }
            f(x, z)
        }
    }
}

/// A partial equivalence relation composes two consecutive steps.
theorem partial_equivalence_step[T](f: (T, T) -> Bool, x: T, y: T, z: T) {
    is_partial_equivalence(f) and f(x, y) and f(y, z) implies f(x, z)
} by {
    if is_partial_equivalence(f) and f(x, y) and f(y, z) {
        partial_equivalence_is_transitive(f)
        is_transitive(f)
        transitive_step(f, x, y, z)
        f(x, z)
    }
}

/// An antisymmetric relation turns two-way comparison into equality.
theorem antisymmetric_eq[T](f: (T, T) -> Bool, x: T, y: T) {
    is_antisymmetric(f) and f(x, y) and f(y, x) implies x = y
} by {
    if is_antisymmetric(f) and f(x, y) and f(y, x) {
        is_antisymmetric(f) = forall(x0: T, y0: T) {
            f(x0, y0) and f(y0, x0) implies x0 = y0
        }
        x = y
    }
}

/// Equality is an equivalence relation.
theorem eq_relation_is_equivalence[T] {
    is_equivalence(eq_relation[T])
} by {
    forall(x: T) {
        eq_relation[T](x, x)
    }
    is_reflexive(eq_relation[T])

    forall(x: T, y: T) {
        if eq_relation[T](x, y) {
            y = x
            eq_relation[T](y, x)
        }
    }
    is_symmetric(eq_relation[T])

    forall(x: T, y: T, z: T) {
        if eq_relation[T](x, y) and eq_relation[T](y, z) {
            x = z
            eq_relation[T](x, z)
        }
    }
    is_transitive(eq_relation[T])

    is_equivalence(eq_relation[T])
}

/// Equality is also a partial equivalence relation.
theorem eq_relation_is_partial_equivalence[T] {
    is_partial_equivalence(eq_relation[T])
} by {
    eq_relation_is_equivalence[T]
    equivalence_is_symmetric(eq_relation[T])
    is_symmetric(eq_relation[T])
    equivalence_is_transitive(eq_relation[T])
    is_transitive(eq_relation[T])
    is_partial_equivalence(eq_relation[T])
}

/// Equality of relations transports reflexivity.
theorem relation_eq_imp_reflexive[T](f: (T, T) -> Bool, g: (T, T) -> Bool) {
    f = g and is_reflexive(f) implies is_reflexive(g)
} by {
    if f = g and is_reflexive(f) {
        binary_function_eq_transport_predicate(is_reflexive[T], f, g)
        is_reflexive(g)
    }
}

/// Equality of relations transports reflexivity in the reverse direction.
theorem relation_eq_imp_reflexive_rev[T](f: (T, T) -> Bool, g: (T, T) -> Bool) {
    f = g and is_reflexive(g) implies is_reflexive(f)
} by {
    if f = g and is_reflexive(g) {
        binary_function_eq_transport_predicate_rev(is_reflexive[T], f, g)
        is_reflexive(f)
    }
}

/// Equality of relations transports symmetry.
theorem relation_eq_imp_symmetric[T](f: (T, T) -> Bool, g: (T, T) -> Bool) {
    f = g and is_symmetric(f) implies is_symmetric(g)
} by {
    if f = g and is_symmetric(f) {
        binary_function_eq_transport_predicate(is_symmetric[T], f, g)
        is_symmetric(g)
    }
}

/// Equality of relations transports symmetry in the reverse direction.
theorem relation_eq_imp_symmetric_rev[T](f: (T, T) -> Bool, g: (T, T) -> Bool) {
    f = g and is_symmetric(g) implies is_symmetric(f)
} by {
    if f = g and is_symmetric(g) {
        binary_function_eq_transport_predicate_rev(is_symmetric[T], f, g)
        is_symmetric(f)
    }
}

/// Equality of relations transports transitivity.
theorem relation_eq_imp_transitive[T](f: (T, T) -> Bool, g: (T, T) -> Bool) {
    f = g and is_transitive(f) implies is_transitive(g)
} by {
    if f = g and is_transitive(f) {
        binary_function_eq_transport_predicate(is_transitive[T], f, g)
        is_transitive(g)
    }
}

/// Equality of relations transports transitivity in the reverse direction.
theorem relation_eq_imp_transitive_rev[T](f: (T, T) -> Bool, g: (T, T) -> Bool) {
    f = g and is_transitive(g) implies is_transitive(f)
} by {
    if f = g and is_transitive(g) {
        binary_function_eq_transport_predicate_rev(is_transitive[T], f, g)
        is_transitive(f)
    }
}

/// Equality of relations transports antisymmetry.
theorem relation_eq_imp_antisymmetric[T](f: (T, T) -> Bool, g: (T, T) -> Bool) {
    f = g and is_antisymmetric(f) implies is_antisymmetric(g)
} by {
    if f = g and is_antisymmetric(f) {
        binary_function_eq_transport_predicate(is_antisymmetric[T], f, g)
        is_antisymmetric(g)
    }
}

/// Equality of relations transports antisymmetry in the reverse direction.
theorem relation_eq_imp_antisymmetric_rev[T](f: (T, T) -> Bool, g: (T, T) -> Bool) {
    f = g and is_antisymmetric(g) implies is_antisymmetric(f)
} by {
    if f = g and is_antisymmetric(g) {
        binary_function_eq_transport_predicate_rev(is_antisymmetric[T], f, g)
        is_antisymmetric(f)
    }
}

/// Equality of relations transports irreflexivity.
theorem relation_eq_imp_irreflexive[T](f: (T, T) -> Bool, g: (T, T) -> Bool) {
    f = g and is_irreflexive(f) implies is_irreflexive(g)
} by {
    if f = g and is_irreflexive(f) {
        binary_function_eq_transport_predicate(is_irreflexive[T], f, g)
        is_irreflexive(g)
    }
}

/// Equality of relations transports irreflexivity in the reverse direction.
theorem relation_eq_imp_irreflexive_rev[T](f: (T, T) -> Bool, g: (T, T) -> Bool) {
    f = g and is_irreflexive(g) implies is_irreflexive(f)
} by {
    if f = g and is_irreflexive(g) {
        binary_function_eq_transport_predicate_rev(is_irreflexive[T], f, g)
        is_irreflexive(f)
    }
}

/// Equality of relations transports asymmetry.
theorem relation_eq_imp_asymmetric[T](f: (T, T) -> Bool, g: (T, T) -> Bool) {
    f = g and is_asymmetric(f) implies is_asymmetric(g)
} by {
    if f = g and is_asymmetric(f) {
        binary_function_eq_transport_predicate(is_asymmetric[T], f, g)
        is_asymmetric(g)
    }
}

/// Equality of relations transports asymmetry in the reverse direction.
theorem relation_eq_imp_asymmetric_rev[T](f: (T, T) -> Bool, g: (T, T) -> Bool) {
    f = g and is_asymmetric(g) implies is_asymmetric(f)
} by {
    if f = g and is_asymmetric(g) {
        binary_function_eq_transport_predicate_rev(is_asymmetric[T], f, g)
        is_asymmetric(f)
    }
}

/// Equality of relations transports totality.
theorem relation_eq_imp_total[T](f: (T, T) -> Bool, g: (T, T) -> Bool) {
    f = g and is_total(f) implies is_total(g)
} by {
    if f = g and is_total(f) {
        binary_function_eq_transport_predicate(is_total[T], f, g)
        is_total(g)
    }
}

/// Equality of relations transports totality in the reverse direction.
theorem relation_eq_imp_total_rev[T](f: (T, T) -> Bool, g: (T, T) -> Bool) {
    f = g and is_total(g) implies is_total(f)
} by {
    if f = g and is_total(g) {
        binary_function_eq_transport_predicate_rev(is_total[T], f, g)
        is_total(f)
    }
}

/// Equality of relations transports equivalence-relation structure.
theorem relation_eq_imp_equivalence[T](f: (T, T) -> Bool, g: (T, T) -> Bool) {
    f = g and is_equivalence(f) implies is_equivalence(g)
} by {
    if f = g and is_equivalence(f) {
        binary_function_eq_transport_predicate(is_equivalence[T], f, g)
        is_equivalence(g)
    }
}

/// Equality of relations transports equivalence-relation structure in the reverse direction.
theorem relation_eq_imp_equivalence_rev[T](f: (T, T) -> Bool, g: (T, T) -> Bool) {
    f = g and is_equivalence(g) implies is_equivalence(f)
} by {
    if f = g and is_equivalence(g) {
        binary_function_eq_transport_predicate_rev(is_equivalence[T], f, g)
        is_equivalence(f)
    }
}

/// Equality of relations transports partial-equivalence structure.
theorem relation_eq_imp_partial_equivalence[T](f: (T, T) -> Bool, g: (T, T) -> Bool) {
    f = g and is_partial_equivalence(f) implies is_partial_equivalence(g)
} by {
    if f = g and is_partial_equivalence(f) {
        binary_function_eq_transport_predicate(is_partial_equivalence[T], f, g)
        is_partial_equivalence(g)
    }
}

/// Equality of relations transports partial-equivalence structure in the reverse direction.
theorem relation_eq_imp_partial_equivalence_rev[T](f: (T, T) -> Bool, g: (T, T) -> Bool) {
    f = g and is_partial_equivalence(g) implies is_partial_equivalence(f)
} by {
    if f = g and is_partial_equivalence(g) {
        binary_function_eq_transport_predicate_rev(is_partial_equivalence[T], f, g)
        is_partial_equivalence(f)
    }
}

/// Equality of a relation source transports relation inclusion.
theorem relation_subset_eq_left[A, B](r: (A, B) -> Bool, s: (A, B) -> Bool, t: (A, B) -> Bool) {
    r = s and relation_subset(r, t) implies relation_subset(s, t)
} by {
    if r = s and relation_subset(r, t) {
        relation_subset(s, t)
    }
}

/// Equality of a relation target transports relation inclusion.
theorem relation_subset_eq_right[A, B](r: (A, B) -> Bool, s: (A, B) -> Bool, t: (A, B) -> Bool) {
    s = t and relation_subset(r, s) implies relation_subset(r, t)
} by {
    if s = t and relation_subset(r, s) {
        relation_subset(r, t)
    }
}

/// Equality of both relation arguments transports relation inclusion.
theorem relation_subset_eq[A, B](r: (A, B) -> Bool, s: (A, B) -> Bool, t: (A, B) -> Bool, u: (A, B) -> Bool) {
    r = s and t = u and relation_subset(r, t) implies relation_subset(s, u)
} by {
    if r = s and t = u and relation_subset(r, t) {
        relation_subset(s, u)
    }
}

/// Reflexivity, symmetry, and transitivity together give an equivalence relation.
theorem reflexive_symmetric_transitive_imp_equivalence[T](f: (T, T) -> Bool) {
    is_reflexive(f) and is_symmetric(f) and is_transitive(f) implies is_equivalence(f)
} by {
    if is_reflexive(f) and is_symmetric(f) and is_transitive(f) {
        is_equivalence(f) = is_reflexive(f) and is_symmetric(f) and is_transitive(f)
        is_equivalence(f)
    }
}

/// Symmetry and transitivity together give a partial equivalence relation.
theorem symmetric_transitive_imp_partial_equivalence[T](f: (T, T) -> Bool) {
    is_symmetric(f) and is_transitive(f) implies is_partial_equivalence(f)
} by {
    if is_symmetric(f) and is_transitive(f) {
        is_partial_equivalence(f) = is_symmetric(f) and is_transitive(f)
        is_partial_equivalence(f)
    }
}

/// Every equivalence relation is a partial equivalence relation.
theorem equivalence_imp_partial_equivalence[T](f: (T, T) -> Bool) {
    is_equivalence(f) implies is_partial_equivalence(f)
} by {
    if is_equivalence(f) {
        equivalence_is_symmetric(f)
        is_symmetric(f)
        equivalence_is_transitive(f)
        is_transitive(f)
        symmetric_transitive_imp_partial_equivalence(f)
        is_partial_equivalence(f)
    }
}

/// An equivalence relation relates every element to itself.
theorem equivalence_self[T](f: (T, T) -> Bool, x: T) {
    is_equivalence(f) implies f(x, x)
} by {
    if is_equivalence(f) {
        equivalence_is_reflexive(f)
        is_reflexive(f)
        reflexive_self(f, x)
        f(x, x)
    }
}

/// An equivalence relation can reverse any related pair.
theorem equivalence_flip[T](f: (T, T) -> Bool, x: T, y: T) {
    is_equivalence(f) and f(x, y) implies f(y, x)
} by {
    if is_equivalence(f) and f(x, y) {
        equivalence_is_symmetric(f)
        is_symmetric(f)
        symmetric_flip(f, x, y)
        f(y, x)
    }
}

/// An equivalence relation composes two related steps.
theorem equivalence_step[T](f: (T, T) -> Bool, x: T, y: T, z: T) {
    is_equivalence(f) and f(x, y) and f(y, z) implies f(x, z)
} by {
    if is_equivalence(f) and f(x, y) and f(y, z) {
        equivalence_is_transitive(f)
        is_transitive(f)
        transitive_step(f, x, y, z)
        f(x, z)
    }
}

/// Relation intersection preserves reflexivity.
theorem relation_intersection_is_reflexive[T](r: (T, T) -> Bool, s: (T, T) -> Bool) {
    is_reflexive(r) and is_reflexive(s) implies is_reflexive(relation_intersection(r, s))
} by {
    if is_reflexive(r) and is_reflexive(s) {
        forall(x: T) {
            reflexive_self(r, x)
            reflexive_self(s, x)
            r(x, x) and s(x, x)
            relation_intersection(r, s, x, x)
        }
    }
}

/// Relation intersection preserves symmetry.
theorem relation_intersection_is_symmetric[T](r: (T, T) -> Bool, s: (T, T) -> Bool) {
    is_symmetric(r) and is_symmetric(s) implies is_symmetric(relation_intersection(r, s))
} by {
    if is_symmetric(r) and is_symmetric(s) {
        forall(x: T, y: T) {
            if relation_intersection(r, s, x, y) {
                r(x, y)
                symmetric_flip(r, x, y)
                r(y, x)
                s(x, y)
                symmetric_flip(s, x, y)
                s(y, x)
                r(y, x) and s(y, x)
                relation_intersection(r, s, y, x)
            }
        }
    }
}

/// Relation intersection preserves transitivity.
theorem relation_intersection_is_transitive[T](r: (T, T) -> Bool, s: (T, T) -> Bool) {
    is_transitive(r) and is_transitive(s) implies is_transitive(relation_intersection(r, s))
} by {
    if is_transitive(r) and is_transitive(s) {
        forall(x: T, y: T, z: T) {
            if relation_intersection(r, s, x, y) and relation_intersection(r, s, y, z) {
                r(x, y)
                r(y, z)
                transitive_step(r, x, y, z)
                r(x, z)
                s(x, y)
                s(y, z)
                transitive_step(s, x, y, z)
                s(x, z)
                r(x, z) and s(x, z)
                relation_intersection(r, s, x, z)
            }
        }
    }
}

/// Relation intersection preserves equivalence relations.
theorem relation_intersection_is_equivalence[T](r: (T, T) -> Bool, s: (T, T) -> Bool) {
    is_equivalence(r) and is_equivalence(s) implies is_equivalence(relation_intersection(r, s))
} by {
    if is_equivalence(r) and is_equivalence(s) {
        equivalence_is_reflexive(r)
        equivalence_is_reflexive(s)
        relation_intersection_is_reflexive(r, s)
        is_reflexive(relation_intersection(r, s))
        equivalence_is_symmetric(r)
        equivalence_is_symmetric(s)
        relation_intersection_is_symmetric(r, s)
        is_symmetric(relation_intersection(r, s))
        equivalence_is_transitive(r)
        equivalence_is_transitive(s)
        relation_intersection_is_transitive(r, s)
        is_transitive(relation_intersection(r, s))
        is_reflexive(relation_intersection(r, s)) and
            is_symmetric(relation_intersection(r, s)) and
            is_transitive(relation_intersection(r, s))
        is_equivalence(relation_intersection(r, s)) =
            is_reflexive(relation_intersection(r, s)) and
            is_symmetric(relation_intersection(r, s)) and
            is_transitive(relation_intersection(r, s))
        is_equivalence(relation_intersection(r, s))
    }
}

/// Relation intersection preserves partial equivalence relations.
theorem relation_intersection_is_partial_equivalence[T](r: (T, T) -> Bool, s: (T, T) -> Bool) {
    is_partial_equivalence(r) and is_partial_equivalence(s) implies
    is_partial_equivalence(relation_intersection(r, s))
} by {
    if is_partial_equivalence(r) and is_partial_equivalence(s) {
        partial_equivalence_is_symmetric(r)
        partial_equivalence_is_symmetric(s)
        relation_intersection_is_symmetric(r, s)
        is_symmetric(relation_intersection(r, s))
        partial_equivalence_is_transitive(r)
        partial_equivalence_is_transitive(s)
        relation_intersection_is_transitive(r, s)
        is_transitive(relation_intersection(r, s))
        is_symmetric(relation_intersection(r, s)) and is_transitive(relation_intersection(r, s))
        is_partial_equivalence(relation_intersection(r, s)) =
            is_symmetric(relation_intersection(r, s)) and is_transitive(relation_intersection(r, s))
        is_partial_equivalence(relation_intersection(r, s))
    }
}

/// Antisymmetry of the left factor passes to the intersection.
theorem relation_intersection_is_antisymmetric_left[T](r: (T, T) -> Bool, s: (T, T) -> Bool) {
    is_antisymmetric(r) implies is_antisymmetric(relation_intersection(r, s))
} by {
    if is_antisymmetric(r) {
        forall(x: T, y: T) {
            if relation_intersection(r, s, x, y) and relation_intersection(r, s, y, x) {
                r(x, y)
                r(y, x)
                antisymmetric_eq(r, x, y)
                x = y
            }
        }
    }
}

/// Antisymmetry of the right factor passes to the intersection.
theorem relation_intersection_is_antisymmetric_right[T](r: (T, T) -> Bool, s: (T, T) -> Bool) {
    is_antisymmetric(s) implies is_antisymmetric(relation_intersection(r, s))
} by {
    if is_antisymmetric(s) {
        forall(x: T, y: T) {
            if relation_intersection(r, s, x, y) and relation_intersection(r, s, y, x) {
                s(x, y)
                s(y, x)
                antisymmetric_eq(s, x, y)
                x = y
            }
        }
    }
}

/// Irreflexivity of the left factor passes to the intersection.
theorem relation_intersection_is_irreflexive_left[T](r: (T, T) -> Bool, s: (T, T) -> Bool) {
    is_irreflexive(r) implies is_irreflexive(relation_intersection(r, s))
} by {
    if is_irreflexive(r) {
        forall(x: T) {
            if relation_intersection(r, s, x, x) {
                r(x, x)
                false
            }
        }
    }
}

/// Irreflexivity of the right factor passes to the intersection.
theorem relation_intersection_is_irreflexive_right[T](r: (T, T) -> Bool, s: (T, T) -> Bool) {
    is_irreflexive(s) implies is_irreflexive(relation_intersection(r, s))
} by {
    if is_irreflexive(s) {
        forall(x: T) {
            if relation_intersection(r, s, x, x) {
                s(x, x)
                false
            }
        }
    }
}

/// Asymmetry of the left factor passes to the intersection.
theorem relation_intersection_is_asymmetric_left[T](r: (T, T) -> Bool, s: (T, T) -> Bool) {
    is_asymmetric(r) implies is_asymmetric(relation_intersection(r, s))
} by {
    if is_asymmetric(r) {
        is_asymmetric(r) = forall(x0: T, y0: T) {
            r(x0, y0) implies not r(y0, x0)
        }
        forall(x: T, y: T) {
            if relation_intersection(r, s, x, y) {
                r(x, y)
                not r(y, x)
                if relation_intersection(r, s, y, x) {
                    r(y, x)
                    false
                }
            }
        }
    }
}

/// Asymmetry of the right factor passes to the intersection.
theorem relation_intersection_is_asymmetric_right[T](r: (T, T) -> Bool, s: (T, T) -> Bool) {
    is_asymmetric(s) implies is_asymmetric(relation_intersection(r, s))
} by {
    if is_asymmetric(s) {
        is_asymmetric(s) = forall(x0: T, y0: T) {
            s(x0, y0) implies not s(y0, x0)
        }
        forall(x: T, y: T) {
            if relation_intersection(r, s, x, y) {
                s(x, y)
                not s(y, x)
                if relation_intersection(r, s, y, x) {
                    s(y, x)
                    false
                }
            }
        }
    }
}

/// Relation union preserves reflexivity.
theorem relation_union_is_reflexive[T](r: (T, T) -> Bool, s: (T, T) -> Bool) {
    is_reflexive(r) and is_reflexive(s) implies is_reflexive(relation_union(r, s))
} by {
    if is_reflexive(r) and is_reflexive(s) {
        forall(x: T) {
            reflexive_self(r, x)
            r(x, x)
            relation_union(r, s, x, x)
        }
    }
}

/// Relation union preserves irreflexivity.
theorem relation_union_is_irreflexive[T](r: (T, T) -> Bool, s: (T, T) -> Bool) {
    is_irreflexive(r) and is_irreflexive(s) implies is_irreflexive(relation_union(r, s))
} by {
    if is_irreflexive(r) and is_irreflexive(s) {
        forall(x: T) {
            if relation_union(r, s, x, x) {
                relation_union(r, s, x, x) = (r(x, x) or s(x, x))
                if r(x, x) {
                    false
                } else {
                    s(x, x)
                    false
                }
            }
        }
    }
}

/// Relation union preserves symmetry.
theorem relation_union_is_symmetric[T](r: (T, T) -> Bool, s: (T, T) -> Bool) {
    is_symmetric(r) and is_symmetric(s) implies is_symmetric(relation_union(r, s))
} by {
    if is_symmetric(r) and is_symmetric(s) {
        forall(x: T, y: T) {
            if relation_union(r, s, x, y) {
                relation_union(r, s, x, y) = (r(x, y) or s(x, y))
                if r(x, y) {
                    symmetric_flip(r, x, y)
                    r(y, x)
                    relation_union(r, s, y, x)
                } else {
                    symmetric_flip(s, x, y)
                    s(y, x)
                    relation_union(r, s, y, x)
                }
            }
        }
    }
}

/// Relation union preserves totality.
theorem relation_union_is_total[T](r: (T, T) -> Bool, s: (T, T) -> Bool) {
    is_total(r) and is_total(s) implies is_total(relation_union(r, s))
} by {
    if is_total(r) and is_total(s) {
        forall(x: T, y: T) {
            is_total(r) = forall(x0: T, y0: T) {
                r(x0, y0) or r(y0, x0)
            }
            r(x, y) or r(y, x)
            if r(x, y) {
                relation_union(r, s, x, y)
                relation_union(r, s, x, y) or relation_union(r, s, y, x)
            } else {
                relation_union(r, s, y, x)
                relation_union(r, s, x, y) or relation_union(r, s, y, x)
            }
        }
    }
}

/// Reflexivity of the left factor extends to the union.
theorem relation_union_is_reflexive_left[T](r: (T, T) -> Bool, s: (T, T) -> Bool) {
    is_reflexive(r) implies is_reflexive(relation_union(r, s))
} by {
    if is_reflexive(r) {
        forall(x: T) {
            reflexive_self(r, x)
            r(x, x)
            relation_union(r, s, x, x)
        }
    }
}

/// Reflexivity of the right factor extends to the union.
theorem relation_union_is_reflexive_right[T](r: (T, T) -> Bool, s: (T, T) -> Bool) {
    is_reflexive(s) implies is_reflexive(relation_union(r, s))
} by {
    if is_reflexive(s) {
        forall(x: T) {
            reflexive_self(s, x)
            s(x, x)
            relation_union(r, s, x, x)
        }
    }
}

/// Totality of the left factor extends to the union.
theorem relation_union_is_total_left[T](r: (T, T) -> Bool, s: (T, T) -> Bool) {
    is_total(r) implies is_total(relation_union(r, s))
} by {
    if is_total(r) {
        is_total(r) = forall(x0: T, y0: T) {
            r(x0, y0) or r(y0, x0)
        }
        forall(x: T, y: T) {
            r(x, y) or r(y, x)
            if r(x, y) {
                relation_union(r, s, x, y)
                relation_union(r, s, x, y) or relation_union(r, s, y, x)
            } else {
                relation_union(r, s, y, x)
                relation_union(r, s, x, y) or relation_union(r, s, y, x)
            }
        }
    }
}

/// Totality of the right factor extends to the union.
theorem relation_union_is_total_right[T](r: (T, T) -> Bool, s: (T, T) -> Bool) {
    is_total(s) implies is_total(relation_union(r, s))
} by {
    if is_total(s) {
        is_total(s) = forall(x0: T, y0: T) {
            s(x0, y0) or s(y0, x0)
        }
        forall(x: T, y: T) {
            s(x, y) or s(y, x)
            if s(x, y) {
                relation_union(r, s, x, y)
                relation_union(r, s, x, y) or relation_union(r, s, y, x)
            } else {
                relation_union(r, s, y, x)
                relation_union(r, s, x, y) or relation_union(r, s, y, x)
            }
        }
    }
}

/// Converse preserves equivalence relations.
theorem relation_converse_is_equivalence[T](r: (T, T) -> Bool) {
    is_equivalence(r) implies is_equivalence(relation_converse(r))
} by {
    if is_equivalence(r) {
        equivalence_is_reflexive(r)
        relation_converse_is_reflexive(r)
        is_reflexive(relation_converse(r))
        equivalence_is_symmetric(r)
        relation_converse_is_symmetric(r)
        is_symmetric(relation_converse(r))
        equivalence_is_transitive(r)
        relation_converse_is_transitive(r)
        is_transitive(relation_converse(r))
        is_equivalence(relation_converse(r))
    }
}

/// Converse preserves partial equivalence relations.
theorem relation_converse_is_partial_equivalence[T](r: (T, T) -> Bool) {
    is_partial_equivalence(r) implies is_partial_equivalence(relation_converse(r))
} by {
    if is_partial_equivalence(r) {
        partial_equivalence_is_symmetric(r)
        relation_converse_is_symmetric(r)
        is_symmetric(relation_converse(r))
        partial_equivalence_is_transitive(r)
        relation_converse_is_transitive(r)
        is_transitive(relation_converse(r))
        is_partial_equivalence(relation_converse(r))
    }
}
