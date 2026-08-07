from nat import Nat, add_zero_right, add_zero_left, add_suc_right, add_suc_left, zero_or_suc
from data.basic.functions import binary_function_extensionality, is_surjective_fn, is_bijection_fn,
    bijection_fn_is_injective, bijection_fn_is_surjective, surjective_fn_has_preimage
from data.basic.relation_basic import eq_relation, relation_compose, relation_compose_intro,
    relation_compose_has_witness, relation_compose_assoc, relation_compose_eq_relation_left,
    relation_compose_eq_relation_right, relation_converse, relation_converse_compose,
    relation_converse_eq_relation, relation_subset, relation_subset_step, reflexive_self,
    transitive_step, is_reflexive, is_transitive, relation_eq_iff_subset_both,
    is_symmetric, is_equivalence, relation_symmetric_closure, relation_subset_symmetric_closure,
    relation_symmetric_closure_subset_of_symmetric, relation_symmetric_closure_monotone,
    relation_symmetric_closure_is_symmetric, symmetric_imp_relation_converse_eq
from data.basic.relation_transport import relation_pullback, relation_pullback_eq_relation_of_injective

/// General constructions on homogeneous relations indexed by natural numbers.

/// The `n`th power of a homogeneous relation under relational composition.
define relation_power[T](r: (T, T) -> Bool, n: Nat, x: T, y: T) -> Bool {
    match n {
        Nat.zero {
            eq_relation[T](x, y)
        }
        Nat.suc(pred) {
            relation_compose(relation_power(r, pred), r, x, y)
        }
    }
}

/// The reflexive-transitive closure of a homogeneous relation, represented by finite relation powers.
define relation_refl_trans_closure[T](r: (T, T) -> Bool, x: T, y: T) -> Bool {
    exists(n: Nat) {
        relation_power(r, n, x, y)
    }
}

/// The transitive closure of a homogeneous relation, represented by positive relation powers.
define relation_trans_closure[T](r: (T, T) -> Bool, x: T, y: T) -> Bool {
    exists(n: Nat) {
        relation_power(r, n.suc, x, y)
    }
}

/// The zeroth relation power is the equality relation.
theorem relation_power_zero[T](r: (T, T) -> Bool) {
    relation_power(r, Nat.zero) = eq_relation[T]
} by {
    let u = relation_power(r, Nat.zero)
    let v = eq_relation[T]
    forall(x: T, y: T) {
        u(x, y) = eq_relation[T](x, y)
        v(x, y) = eq_relation[T](x, y)
        u(x, y) = v(x, y)
    }
    binary_function_extensionality(u, v)
}

/// The successor relation power is one more composition by the original relation.
theorem relation_power_suc[T](r: (T, T) -> Bool, n: Nat) {
    relation_power(r, n.suc) = relation_compose(relation_power(r, n), r)
} by {
    let u = relation_power(r, n.suc)
    let v = relation_compose(relation_power(r, n), r)
    forall(x: T, y: T) {
        u(x, y) = relation_compose(relation_power(r, n), r, x, y)
        v(x, y) = relation_compose(relation_power(r, n), r, x, y)
        u(x, y) = v(x, y)
    }
    binary_function_extensionality(u, v)
}

/// The first positive relation power is the original relation.
theorem relation_power_one[T](r: (T, T) -> Bool) {
    relation_power(r, Nat.zero.suc) = r
} by {
    relation_power_suc(r, Nat.zero)
    relation_power_zero(r)
    relation_compose_eq_relation_left(r)
}

/// The zeroth relation power is reflexive.
theorem relation_power_zero_self[T](r: (T, T) -> Bool, x: T) {
    relation_power(r, Nat.zero, x, x)
} by {
    eq_relation[T](x, x)
    relation_power(r, Nat.zero, x, x)
}

/// One step of the original relation belongs to the first positive relation power.
theorem relation_power_one_step[T](r: (T, T) -> Bool, x: T, y: T) {
    r(x, y) implies relation_power(r, Nat.zero.suc, x, y)
} by {
    if r(x, y) {
        relation_power_zero_self(r, x)
        relation_compose_intro(relation_power(r, Nat.zero), r, x, x, y)
        relation_power(r, Nat.zero.suc, x, y)
    }
}

/// Any finite relation power belongs to the reflexive-transitive closure.
theorem relation_power_in_refl_trans_closure[T](r: (T, T) -> Bool, n: Nat, x: T, y: T) {
    relation_power(r, n, x, y) implies relation_refl_trans_closure(r, x, y)
} by {
    if relation_power(r, n, x, y) {
        exists(m: Nat) {
            m = n and relation_power(r, m, x, y)
        }
        relation_refl_trans_closure(r, x, y)
    }
}

/// Any positive relation power belongs to the transitive closure.
theorem relation_power_positive_in_trans_closure[T](r: (T, T) -> Bool, n: Nat, x: T, y: T) {
    relation_power(r, n.suc, x, y) implies relation_trans_closure(r, x, y)
} by {
    if relation_power(r, n.suc, x, y) {
        exists(m: Nat) {
            m = n and relation_power(r, m.suc, x, y)
        }
        relation_trans_closure(r, x, y)
    }
}

/// The original relation is contained in its reflexive-transitive closure.
theorem relation_subset_refl_trans_closure[T](r: (T, T) -> Bool) {
    relation_subset(r, relation_refl_trans_closure(r))
} by {
    forall(x: T, y: T) {
        if r(x, y) {
            relation_power_one_step(r, x, y)
            relation_power_in_refl_trans_closure(r, Nat.zero.suc, x, y)
            relation_refl_trans_closure(r, x, y)
        }
    }
}

/// Reflexive-transitive closure is reflexive.
theorem relation_refl_trans_closure_is_reflexive[T](r: (T, T) -> Bool) {
    is_reflexive(relation_refl_trans_closure(r))
} by {
    forall(x: T) {
        relation_power_zero_self(r, x)
        relation_power_in_refl_trans_closure(r, Nat.zero, x, x)
        relation_refl_trans_closure(r, x, x)
    }
}

/// The original relation is contained in its transitive closure.
theorem relation_subset_trans_closure[T](r: (T, T) -> Bool) {
    relation_subset(r, relation_trans_closure(r))
} by {
    forall(x: T, y: T) {
        if r(x, y) {
            relation_power_one_step(r, x, y)
            relation_power_positive_in_trans_closure(r, Nat.zero, x, y)
            relation_trans_closure(r, x, y)
        }
    }
}

/// Transitive closure is contained in reflexive-transitive closure.
theorem relation_trans_closure_subset_refl_trans_closure[T](r: (T, T) -> Bool) {
    relation_subset(relation_trans_closure(r), relation_refl_trans_closure(r))
} by {
    forall(x: T, y: T) {
        if relation_trans_closure(r, x, y) {
            let n: Nat satisfy {
                relation_power(r, n.suc, x, y)
            }
            exists(m: Nat) {
                m = n.suc and relation_power(r, m, x, y)
            }
            relation_refl_trans_closure(r, x, y)
        }
    }
}

/// Reflexive-transitive closure splits into equality or positive-length reachability.
theorem relation_refl_trans_closure_eq_eq_or_trans_closure[T](r: (T, T) -> Bool, x: T, y: T) {
    relation_refl_trans_closure(r, x, y) = (eq_relation[T](x, y) or relation_trans_closure(r, x, y))
} by {
    if relation_refl_trans_closure(r, x, y) {
        let n: Nat satisfy {
            relation_power(r, n, x, y)
        }
        zero_or_suc(n)
        if n = Nat.zero {
            relation_power(r, Nat.zero, x, y)
            relation_power_zero(r)
            eq_relation[T](x, y)
        } else {
            let k: Nat satisfy {
                n = k.suc
            }
            relation_power(r, k.suc, x, y)
            relation_power_positive_in_trans_closure(r, k, x, y)
            relation_trans_closure(r, x, y)
        }
        eq_relation[T](x, y) or relation_trans_closure(r, x, y)
    }
    if eq_relation[T](x, y) or relation_trans_closure(r, x, y) {
        if eq_relation[T](x, y) {
            relation_power_zero(r)
            relation_power(r, Nat.zero, x, y)
            relation_power_in_refl_trans_closure(r, Nat.zero, x, y)
            relation_refl_trans_closure(r, x, y)
        } else {
            relation_trans_closure_subset_refl_trans_closure(r)
            relation_refl_trans_closure(r, x, y)
        }
    }
}

/// The first successor-step rewrite for the additive law on relation powers.
theorem relation_power_add_step_compose[T](r: (T, T) -> Bool, m: Nat, k: Nat) {
    relation_compose(relation_power(r, m), relation_power(r, k)) = relation_power(r, m + k)
    implies
    relation_compose(relation_power(r, m), relation_power(r, k.suc)) = relation_compose(relation_power(r, m + k), r)
} by {
    if relation_compose(relation_power(r, m), relation_power(r, k)) = relation_power(r, m + k) {
        relation_power_suc(r, k)
        relation_compose_assoc(relation_power(r, m), relation_power(r, k), r)
    }
}

/// The second successor-step rewrite for the additive law on relation powers.
theorem relation_power_add_step_power[T](r: (T, T) -> Bool, m: Nat, k: Nat) {
    relation_compose(relation_power(r, m + k), r) = relation_power(r, m + k.suc)
} by {
    relation_power_suc(r, m + k)
    add_suc_right(m, k)
}

/// Relation powers compose by adding their exponents.
theorem relation_power_add[T](r: (T, T) -> Bool, m: Nat, n: Nat) {
    relation_compose(relation_power(r, m), relation_power(r, n)) = relation_power(r, m + n)
} by {
    define p(k: Nat) -> Bool {
        relation_compose(relation_power(r, m), relation_power(r, k)) = relation_power(r, m + k)
    }
    relation_power_zero(r)
    relation_compose_eq_relation_right(relation_power(r, m))
    add_zero_right(m)
    p(Nat.zero)
    forall(k: Nat) {
        if p(k) {
            relation_power_add_step_compose(r, m, k)
            relation_power_add_step_power(r, m, k)
            p(k.suc)
        }
    }
    p(n)
}

/// The successor-step monotonicity law for relation powers.
theorem relation_power_monotone_step[T](r: (T, T) -> Bool, s: (T, T) -> Bool, n: Nat) {
    relation_subset(r, s) and relation_subset(relation_power(r, n), relation_power(s, n))
    implies
    relation_subset(relation_power(r, n.suc), relation_power(s, n.suc))
} by {
    if relation_subset(r, s) and relation_subset(relation_power(r, n), relation_power(s, n)) {
        forall(x: T, y: T) {
            if relation_power(r, n.suc, x, y) {
                relation_power_suc(r, n)
                relation_compose_has_witness(relation_power(r, n), r, x, y)
                let z: T satisfy {
                    relation_power(r, n, x, z) and r(z, y)
                }
                relation_subset_step(relation_power(r, n), relation_power(s, n), x, z)
                relation_power(s, n, x, z)
                relation_subset_step(r, s, z, y)
                s(z, y)
                relation_compose_intro(relation_power(s, n), s, x, z, y)
                relation_power_suc(s, n)
                relation_power(s, n.suc, x, y)
            }
        }
    }
}

/// Relation powers are monotone in the base relation.
theorem relation_power_monotone[T](r: (T, T) -> Bool, s: (T, T) -> Bool, n: Nat) {
    relation_subset(r, s) implies relation_subset(relation_power(r, n), relation_power(s, n))
} by {
    define p(k: Nat) -> Bool {
        relation_subset(r, s) implies relation_subset(relation_power(r, k), relation_power(s, k))
    }
    if relation_subset(r, s) {
        forall(x: T, y: T) {
            if relation_power(r, Nat.zero, x, y) {
                relation_power_zero(r)
                eq_relation[T](x, y)
                relation_power_zero(s)
                relation_power(s, Nat.zero, x, y)
            }
        }
        relation_subset(relation_power(r, Nat.zero), relation_power(s, Nat.zero))
        p(Nat.zero)
    }
    forall(k: Nat) {
        if p(k) {
            if relation_subset(r, s) {
                p(k)
                relation_subset(relation_power(r, k), relation_power(s, k))
                relation_power_monotone_step(r, s, k)
                relation_subset(relation_power(r, k.suc), relation_power(s, k.suc))
                p(k.suc)
            }
        }
    }
    p(n)
}

/// One more step on the left also yields the successor relation power.
theorem relation_power_suc_left[T](r: (T, T) -> Bool, n: Nat) {
    relation_compose(r, relation_power(r, n)) = relation_power(r, n.suc)
} by {
    relation_power_one(r)
    relation_power_add(r, Nat.zero.suc, n)
    add_suc_left(Nat.zero, n)
    add_zero_left(n)
}

/// The converse of a successor relation power becomes a left composition.
theorem relation_power_converse_suc_compose[T](r: (T, T) -> Bool, k: Nat) {
    relation_converse(relation_power(r, k.suc)) = relation_compose(relation_converse(r), relation_converse(relation_power(r, k)))
} by {
    relation_power_suc(r, k)
    relation_converse_compose(relation_power(r, k), r)
}

/// A left composition by the converse relation is the next converse power.
theorem relation_power_converse_suc_target[T](r: (T, T) -> Bool, k: Nat) {
    relation_compose(relation_converse(r), relation_power(relation_converse(r), k)) = relation_power(relation_converse(r), k.suc)
} by {
    relation_power_suc_left(relation_converse(r), k)
}

/// The successor-step converse law for relation powers.
theorem relation_power_converse_step[T](r: (T, T) -> Bool, k: Nat) {
    relation_converse(relation_power(r, k)) = relation_power(relation_converse(r), k)
    implies
    relation_converse(relation_power(r, k.suc)) = relation_power(relation_converse(r), k.suc)
} by {
    if relation_converse(relation_power(r, k)) = relation_power(relation_converse(r), k) {
        relation_power_converse_suc_compose(r, k)
        relation_power_converse_suc_target(r, k)
    }
}

/// Taking converse commutes with relation powers.
theorem relation_power_converse[T](r: (T, T) -> Bool, n: Nat) {
    relation_converse(relation_power(r, n)) = relation_power(relation_converse(r), n)
} by {
    define p(k: Nat) -> Bool {
        relation_converse(relation_power(r, k)) = relation_power(relation_converse(r), k)
    }
    relation_power_zero(r)
    relation_converse_eq_relation[T]
    relation_power_zero(relation_converse(r))
    p(Nat.zero)
    forall(k: Nat) {
        if p(k) {
            relation_power_converse_step(r, k)
            p(k.suc)
        }
    }
    p(n)
}

/// One pullback step on relation powers follows from the previous exponent.
theorem relation_power_pullback_subset_step[A, B](f: A -> B, r: (B, B) -> Bool, n: Nat) {
    relation_subset(
        relation_power(relation_pullback(f, r), n),
        relation_pullback(f, relation_power(r, n))
    )
    implies
    relation_subset(
        relation_power(relation_pullback(f, r), n.suc),
        relation_pullback(f, relation_power(r, n.suc))
    )
} by {
    if relation_subset(
        relation_power(relation_pullback(f, r), n),
        relation_pullback(f, relation_power(r, n))
    ) {
        forall(x: A, y: A) {
            if relation_power(relation_pullback(f, r), n.suc, x, y) {
                relation_power_suc(relation_pullback(f, r), n)
                relation_compose_has_witness(relation_power(relation_pullback(f, r), n), relation_pullback(f, r), x, y)
                let z: A satisfy {
                    relation_power(relation_pullback(f, r), n, x, z) and relation_pullback(f, r, z, y)
                }
                relation_subset_step(
                    relation_power(relation_pullback(f, r), n),
                    relation_pullback(f, relation_power(r, n)),
                    x,
                    z
                )
                relation_pullback(f, relation_power(r, n), x, z)
                relation_pullback(f, relation_power(r, n), x, z) = relation_power(r, n, f(x), f(z))
                relation_pullback(f, r, z, y) = r(f(z), f(y))
                relation_compose_intro(relation_power(r, n), r, f(x), f(z), f(y))
                relation_compose(relation_power(r, n), r, f(x), f(y))
                relation_power_suc(r, n)
                relation_power(r, n.suc, f(x), f(y))
                relation_pullback(f, relation_power(r, n.suc), x, y)
            }
        }
    }
}

/// Relation powers under pullback always map into the pullback of the corresponding power.
theorem relation_power_pullback_subset[A, B](f: A -> B, r: (B, B) -> Bool, n: Nat) {
    relation_subset(
        relation_power(relation_pullback(f, r), n),
        relation_pullback(f, relation_power(r, n))
    )
} by {
    define p(k: Nat) -> Bool {
        relation_subset(
            relation_power(relation_pullback(f, r), k),
            relation_pullback(f, relation_power(r, k))
        )
    }
    forall(x: A, y: A) {
        if relation_power(relation_pullback(f, r), Nat.zero, x, y) {
            relation_power_zero(relation_pullback(f, r))
            eq_relation[A](x, y)
            relation_power_zero(r)
            relation_pullback(f, relation_power(r, Nat.zero), x, y) = eq_relation[B](f(x), f(y))
            f(x) = f(y)
            relation_pullback(f, relation_power(r, Nat.zero), x, y)
        }
    }
    relation_subset(
        relation_power(relation_pullback(f, r), Nat.zero),
        relation_pullback(f, relation_power(r, Nat.zero))
    )
    p(Nat.zero)
    forall(k: Nat) {
        if p(k) {
            relation_power_pullback_subset_step(f, r, k)
            p(k.suc)
        }
    }
    p(n)
}

/// The positive-power reverse transport step uses surjectivity to lift one more witness.
theorem relation_power_positive_pullback_reverse_of_surjective_step[A, B](f: A -> B, r: (B, B) -> Bool, n: Nat) {
    is_surjective_fn(f) and
    relation_subset(
        relation_pullback(f, relation_power(r, n.suc)),
        relation_power(relation_pullback(f, r), n.suc)
    )
    implies
    relation_subset(
        relation_pullback(f, relation_power(r, n.suc.suc)),
        relation_power(relation_pullback(f, r), n.suc.suc)
    )
} by {
    if is_surjective_fn(f) and
        relation_subset(
            relation_pullback(f, relation_power(r, n.suc)),
            relation_power(relation_pullback(f, r), n.suc)
        ) {
        forall(x: A, y: A) {
            if relation_pullback(f, relation_power(r, n.suc.suc), x, y) {
                relation_power_suc(r, n.suc)
                relation_pullback(f, relation_power(r, n.suc.suc), x, y) = relation_compose(relation_power(r, n.suc), r, f(x), f(y))
                relation_compose_has_witness(relation_power(r, n.suc), r, f(x), f(y))
                let w: B satisfy {
                    relation_power(r, n.suc, f(x), w) and r(w, f(y))
                }
                surjective_fn_has_preimage(f, w)
                let z: A satisfy {
                    f(z) = w
                }
                relation_pullback(f, relation_power(r, n.suc), x, z) = relation_power(r, n.suc, f(x), f(z))
                f(z) = w
                relation_pullback(f, relation_power(r, n.suc), x, z)
                relation_subset_step(
                    relation_pullback(f, relation_power(r, n.suc)),
                    relation_power(relation_pullback(f, r), n.suc),
                    x,
                    z
                )
                relation_power(relation_pullback(f, r), n.suc, x, z)
                relation_pullback(f, r, z, y) = r(f(z), f(y))
                relation_pullback(f, r, z, y)
                relation_compose_intro(relation_power(relation_pullback(f, r), n.suc), relation_pullback(f, r), x, z, y)
                relation_power_suc(relation_pullback(f, r), n.suc)
                relation_power(relation_pullback(f, r), n.suc.suc, x, y)
            }
        }
    }
}

/// Surjective pullback reflects every positive relation power.
theorem relation_power_positive_pullback_reverse_of_surjective[A, B](f: A -> B, r: (B, B) -> Bool, n: Nat, x: A, y: A) {
    is_surjective_fn(f) and relation_pullback(f, relation_power(r, n.suc), x, y)
    implies
    relation_power(relation_pullback(f, r), n.suc, x, y)
} by {
    define p(k: Nat) -> Bool {
        is_surjective_fn(f) implies
        relation_subset(
            relation_pullback(f, relation_power(r, k.suc)),
            relation_power(relation_pullback(f, r), k.suc)
        )
    }
    if is_surjective_fn(f) {
        forall(a: A, b: A) {
            if relation_pullback(f, relation_power(r, Nat.zero.suc), a, b) {
                relation_power_one(r)
                relation_pullback(f, relation_power(r, Nat.zero.suc), a, b) = relation_pullback(f, r, a, b)
                relation_pullback(f, r, a, b)
                relation_power_one_step(relation_pullback(f, r), a, b)
                relation_power(relation_pullback(f, r), Nat.zero.suc, a, b)
            }
        }
        relation_subset(
            relation_pullback(f, relation_power(r, Nat.zero.suc)),
            relation_power(relation_pullback(f, r), Nat.zero.suc)
        )
        p(Nat.zero)
    }
    forall(k: Nat) {
        if p(k) {
            if is_surjective_fn(f) {
                p(k)
                relation_subset(
                    relation_pullback(f, relation_power(r, k.suc)),
                    relation_power(relation_pullback(f, r), k.suc)
                )
                relation_power_positive_pullback_reverse_of_surjective_step(f, r, k)
                relation_subset(
                    relation_pullback(f, relation_power(r, k.suc.suc)),
                    relation_power(relation_pullback(f, r), k.suc.suc)
                )
                p(k.suc)
            }
        }
    }
    p(n)
    relation_subset(
        relation_pullback(f, relation_power(r, n.suc)),
        relation_power(relation_pullback(f, r), n.suc)
    )
    relation_subset_step(
        relation_pullback(f, relation_power(r, n.suc)),
        relation_power(relation_pullback(f, r), n.suc),
        x,
        y
    )
    relation_power(relation_pullback(f, r), n.suc, x, y)
}

/// Surjective pullback gives exact transport for each positive relation-power instance.
theorem relation_power_positive_pullback_of_surjective_at[A, B](f: A -> B, r: (B, B) -> Bool, n: Nat, x: A, y: A) {
    is_surjective_fn(f) implies
    relation_pullback(f, relation_power(r, n.suc), x, y) = relation_power(relation_pullback(f, r), n.suc, x, y)
} by {
    if is_surjective_fn(f) {
        if relation_pullback(f, relation_power(r, n.suc), x, y) {
            relation_power_positive_pullback_reverse_of_surjective(f, r, n, x, y)
            relation_power(relation_pullback(f, r), n.suc, x, y)
        }
        if relation_power(relation_pullback(f, r), n.suc, x, y) {
            relation_power_pullback_subset(f, r, n.suc)
            relation_subset(
                relation_power(relation_pullback(f, r), n.suc),
                relation_pullback(f, relation_power(r, n.suc))
            )
            relation_subset_step(
                relation_power(relation_pullback(f, r), n.suc),
                relation_pullback(f, relation_power(r, n.suc)),
                x,
                y
            )
            relation_pullback(f, relation_power(r, n.suc), x, y)
        }
        relation_pullback(f, relation_power(r, n.suc), x, y) = relation_power(relation_pullback(f, r), n.suc, x, y)
    }
}

/// Surjective pullback gives exact transport for positive relation powers.
theorem relation_power_positive_pullback_of_surjective[A, B](f: A -> B, r: (B, B) -> Bool, n: Nat) {
    is_surjective_fn(f) implies
    relation_pullback(f, relation_power(r, n.suc)) = relation_power(relation_pullback(f, r), n.suc)
} by {
    let u = relation_pullback(f, relation_power(r, n.suc))
    let v = relation_power(relation_pullback(f, r), n.suc)
    if is_surjective_fn(f) {
        forall(x: A, y: A) {
            relation_power_positive_pullback_of_surjective_at(f, r, n, x, y)
            u(x, y) = v(x, y)
        }
        binary_function_extensionality(u, v)
    }
}

/// Reflexive-transitive closure is monotone in the base relation.
theorem relation_refl_trans_closure_monotone[T](r: (T, T) -> Bool, s: (T, T) -> Bool) {
    relation_subset(r, s) implies relation_subset(relation_refl_trans_closure(r), relation_refl_trans_closure(s))
} by {
    if relation_subset(r, s) {
        forall(x: T, y: T) {
            if relation_refl_trans_closure(r, x, y) {
                let n: Nat satisfy {
                    relation_power(r, n, x, y)
                }
                relation_power_monotone(r, s, n)
                relation_subset(relation_power(r, n), relation_power(s, n))
                relation_subset_step(relation_power(r, n), relation_power(s, n), x, y)
                relation_power(s, n, x, y)
                relation_power_in_refl_trans_closure(s, n, x, y)
                relation_refl_trans_closure(s, x, y)
            }
        }
    }
}

/// Transitive closure is monotone in the base relation.
theorem relation_trans_closure_monotone[T](r: (T, T) -> Bool, s: (T, T) -> Bool) {
    relation_subset(r, s) implies relation_subset(relation_trans_closure(r), relation_trans_closure(s))
} by {
    if relation_subset(r, s) {
        forall(x: T, y: T) {
            if relation_trans_closure(r, x, y) {
                let n: Nat satisfy {
                    relation_power(r, n.suc, x, y)
                }
                relation_power_monotone(r, s, n.suc)
                relation_subset(relation_power(r, n.suc), relation_power(s, n.suc))
                relation_subset_step(relation_power(r, n.suc), relation_power(s, n.suc), x, y)
                relation_power(s, n.suc, x, y)
                relation_power_positive_in_trans_closure(s, n, x, y)
                relation_trans_closure(s, x, y)
            }
        }
    }
}

/// Reflexive-transitive closure under pullback always maps into the pullback of the corresponding closure.
theorem relation_refl_trans_closure_pullback_subset[A, B](f: A -> B, r: (B, B) -> Bool) {
    relation_subset(
        relation_refl_trans_closure(relation_pullback(f, r)),
        relation_pullback(f, relation_refl_trans_closure(r))
    )
} by {
    forall(x: A, y: A) {
        if relation_refl_trans_closure(relation_pullback(f, r), x, y) {
            let n: Nat satisfy {
                relation_power(relation_pullback(f, r), n, x, y)
            }
            relation_power_pullback_subset(f, r, n)
            relation_subset(
                relation_power(relation_pullback(f, r), n),
                relation_pullback(f, relation_power(r, n))
            )
            relation_subset_step(
                relation_power(relation_pullback(f, r), n),
                relation_pullback(f, relation_power(r, n)),
                x,
                y
            )
            relation_pullback(f, relation_power(r, n), x, y)
            relation_pullback(f, relation_power(r, n), x, y) = relation_power(r, n, f(x), f(y))
            relation_power_in_refl_trans_closure(r, n, f(x), f(y))
            relation_refl_trans_closure(r, f(x), f(y))
            relation_pullback(f, relation_refl_trans_closure(r), x, y)
        }
    }
}

/// Transitive closure under pullback always maps into the pullback of the corresponding closure.
theorem relation_trans_closure_pullback_subset[A, B](f: A -> B, r: (B, B) -> Bool) {
    relation_subset(
        relation_trans_closure(relation_pullback(f, r)),
        relation_pullback(f, relation_trans_closure(r))
    )
} by {
    forall(x: A, y: A) {
        if relation_trans_closure(relation_pullback(f, r), x, y) {
            let n: Nat satisfy {
                relation_power(relation_pullback(f, r), n.suc, x, y)
            }
            relation_power_pullback_subset(f, r, n.suc)
            relation_subset(
                relation_power(relation_pullback(f, r), n.suc),
                relation_pullback(f, relation_power(r, n.suc))
            )
            relation_subset_step(
                relation_power(relation_pullback(f, r), n.suc),
                relation_pullback(f, relation_power(r, n.suc)),
                x,
                y
            )
            relation_pullback(f, relation_power(r, n.suc), x, y)
            relation_pullback(f, relation_power(r, n.suc), x, y) = relation_power(r, n.suc, f(x), f(y))
            relation_power_positive_in_trans_closure(r, n, f(x), f(y))
            relation_trans_closure(r, f(x), f(y))
            relation_pullback(f, relation_trans_closure(r), x, y)
        }
    }
}

/// Surjective pullback gives exact transport for each transitive-closure instance.
theorem relation_trans_closure_pullback_of_surjective_at[A, B](f: A -> B, r: (B, B) -> Bool, x: A, y: A) {
    is_surjective_fn(f) implies
    relation_pullback(f, relation_trans_closure(r), x, y) = relation_trans_closure(relation_pullback(f, r), x, y)
} by {
    if is_surjective_fn(f) {
        if relation_pullback(f, relation_trans_closure(r), x, y) {
            let n: Nat satisfy {
                relation_power(r, n.suc, f(x), f(y))
            }
            relation_pullback(f, relation_power(r, n.suc), x, y) = relation_power(r, n.suc, f(x), f(y))
            relation_pullback(f, relation_power(r, n.suc), x, y)
            relation_power_positive_pullback_reverse_of_surjective(f, r, n, x, y)
            relation_power_positive_in_trans_closure(relation_pullback(f, r), n, x, y)
            relation_trans_closure(relation_pullback(f, r), x, y)
        }
        if relation_trans_closure(relation_pullback(f, r), x, y) {
            relation_trans_closure_pullback_subset(f, r)
            relation_subset_step(
                relation_trans_closure(relation_pullback(f, r)),
                relation_pullback(f, relation_trans_closure(r)),
                x,
                y
            )
            relation_pullback(f, relation_trans_closure(r), x, y)
        }
        relation_pullback(f, relation_trans_closure(r), x, y) = relation_trans_closure(relation_pullback(f, r), x, y)
    }
}

/// Surjective pullback gives exact transport for transitive closure.
theorem relation_trans_closure_pullback_of_surjective[A, B](f: A -> B, r: (B, B) -> Bool) {
    is_surjective_fn(f) implies
    relation_pullback(f, relation_trans_closure(r)) = relation_trans_closure(relation_pullback(f, r))
} by {
    let u = relation_pullback(f, relation_trans_closure(r))
    let v = relation_trans_closure(relation_pullback(f, r))
    if is_surjective_fn(f) {
        forall(x: A, y: A) {
            relation_trans_closure_pullback_of_surjective_at(f, r, x, y)
            u(x, y) = v(x, y)
        }
        binary_function_extensionality(u, v)
    }
}

/// Reflexive-transitive closure commutes with converse.
theorem relation_refl_trans_closure_converse[T](r: (T, T) -> Bool, x: T, y: T) {
    relation_refl_trans_closure(relation_converse(r), x, y) = relation_refl_trans_closure(r, y, x)
} by {
    if relation_refl_trans_closure(relation_converse(r), x, y) {
        let n: Nat satisfy {
            relation_power(relation_converse(r), n, x, y)
        }
        relation_power_converse(r, n)
        relation_converse(relation_power(r, n), x, y)
        relation_power(r, n, y, x)
        relation_power_in_refl_trans_closure(r, n, y, x)
        relation_refl_trans_closure(r, y, x)
    }
    if relation_refl_trans_closure(r, y, x) {
        let n: Nat satisfy {
            relation_power(r, n, y, x)
        }
        relation_power_converse(r, n)
        relation_converse(relation_power(r, n), x, y)
        relation_power(relation_converse(r), n, x, y)
        relation_power_in_refl_trans_closure(relation_converse(r), n, x, y)
        relation_refl_trans_closure(relation_converse(r), x, y)
    }
}

/// Transitive closure commutes with converse.
theorem relation_trans_closure_converse[T](r: (T, T) -> Bool, x: T, y: T) {
    relation_trans_closure(relation_converse(r), x, y) = relation_trans_closure(r, y, x)
} by {
    if relation_trans_closure(relation_converse(r), x, y) {
        let n: Nat satisfy {
            relation_power(relation_converse(r), n.suc, x, y)
        }
        relation_power_converse(r, n.suc)
        relation_converse(relation_power(r, n.suc), x, y)
        relation_power(r, n.suc, y, x)
        relation_power_positive_in_trans_closure(r, n, y, x)
        relation_trans_closure(r, y, x)
    }
    if relation_trans_closure(r, y, x) {
        let n: Nat satisfy {
            relation_power(r, n.suc, y, x)
        }
        relation_power_converse(r, n.suc)
        relation_converse(relation_power(r, n.suc), x, y)
        relation_power(relation_converse(r), n.suc, x, y)
        relation_power_positive_in_trans_closure(relation_converse(r), n, x, y)
        relation_trans_closure(relation_converse(r), x, y)
    }
}

/// Bijective pullback gives exact transport for each reflexive-transitive closure instance.
theorem relation_refl_trans_closure_pullback_of_bijection_at[A, B](f: A -> B, r: (B, B) -> Bool, x: A, y: A) {
    is_bijection_fn(f) implies
    relation_pullback(f, relation_refl_trans_closure(r), x, y) = relation_refl_trans_closure(relation_pullback(f, r), x, y)
} by {
    if is_bijection_fn(f) {
        bijection_fn_is_injective(f)
        bijection_fn_is_surjective(f)
        if relation_pullback(f, relation_refl_trans_closure(r), x, y) {
            let n: Nat satisfy {
                relation_power(r, n, f(x), f(y))
            }
            zero_or_suc(n)
            if n = Nat.zero {
                relation_pullback(f, relation_power(r, Nat.zero), x, y) = relation_power(r, Nat.zero, f(x), f(y))
                relation_power_zero(r)
                relation_pullback(f, eq_relation[B], x, y)
                relation_pullback_eq_relation_of_injective(f)
                eq_relation[A](x, y)
                relation_power_zero_self(relation_pullback(f, r), x)
                relation_power_in_refl_trans_closure(relation_pullback(f, r), Nat.zero, x, y)
                relation_refl_trans_closure(relation_pullback(f, r), x, y)
            } else {
                let k: Nat satisfy {
                    n = k.suc
                }
                relation_pullback(f, relation_power(r, k.suc), x, y) = relation_power(r, k.suc, f(x), f(y))
                relation_pullback(f, relation_power(r, k.suc), x, y)
                relation_power_positive_pullback_reverse_of_surjective(f, r, k, x, y)
                relation_power_in_refl_trans_closure(relation_pullback(f, r), k.suc, x, y)
                relation_refl_trans_closure(relation_pullback(f, r), x, y)
            }
        }
        if relation_refl_trans_closure(relation_pullback(f, r), x, y) {
            relation_refl_trans_closure_pullback_subset(f, r)
            relation_subset_step(
                relation_refl_trans_closure(relation_pullback(f, r)),
                relation_pullback(f, relation_refl_trans_closure(r)),
                x,
                y
            )
            relation_pullback(f, relation_refl_trans_closure(r), x, y)
        }
        relation_pullback(f, relation_refl_trans_closure(r), x, y) = relation_refl_trans_closure(relation_pullback(f, r), x, y)
    }
}

/// Bijective pullback gives exact transport for reflexive-transitive closure.
theorem relation_refl_trans_closure_pullback_of_bijection[A, B](f: A -> B, r: (B, B) -> Bool) {
    is_bijection_fn(f) implies
    relation_pullback(f, relation_refl_trans_closure(r)) = relation_refl_trans_closure(relation_pullback(f, r))
} by {
    let u = relation_pullback(f, relation_refl_trans_closure(r))
    let v = relation_refl_trans_closure(relation_pullback(f, r))
    if is_bijection_fn(f) {
        forall(x: A, y: A) {
            relation_refl_trans_closure_pullback_of_bijection_at(f, r, x, y)
            u(x, y) = v(x, y)
        }
        binary_function_extensionality(u, v)
    }
}

/// Any finite relation power lies in every reflexive transitive superrelation containing the base relation.
theorem relation_power_subset_of_reflexive_transitive[T](r: (T, T) -> Bool, s: (T, T) -> Bool, n: Nat) {
    is_reflexive(s) and is_transitive(s) and relation_subset(r, s)
    implies
    relation_subset(relation_power(r, n), s)
} by {
    if is_reflexive(s) and is_transitive(s) and relation_subset(r, s) {
        define p(k: Nat) -> Bool {
            relation_subset(relation_power(r, k), s)
        }
        forall(x: T, y: T) {
            if relation_power(r, Nat.zero, x, y) {
                relation_power_zero(r)
                eq_relation[T](x, y)
                reflexive_self(s, x)
                s(x, y)
            }
        }
        relation_subset(relation_power(r, Nat.zero), s)
        p(Nat.zero)
        forall(k: Nat) {
            if p(k) {
                p(k)
                relation_subset(relation_power(r, k), s)
                forall(x: T, y: T) {
                    if relation_power(r, k.suc, x, y) {
                        relation_power_suc(r, k)
                        relation_compose_has_witness(relation_power(r, k), r, x, y)
                        let z: T satisfy {
                            relation_power(r, k, x, z) and r(z, y)
                        }
                        relation_subset_step(relation_power(r, k), s, x, z)
                        s(x, z)
                        relation_subset_step(r, s, z, y)
                        s(z, y)
                        transitive_step(s, x, z, y)
                        s(x, y)
                    }
                }
                relation_subset(relation_power(r, k.suc), s)
                p(k.suc)
            }
        }
        p(n)
        relation_subset(relation_power(r, n), s)
    }
}

/// Any positive relation power lies in every transitive superrelation containing the base relation.
theorem relation_power_positive_subset_of_transitive[T](r: (T, T) -> Bool, s: (T, T) -> Bool, n: Nat) {
    is_transitive(s) and relation_subset(r, s)
    implies
    relation_subset(relation_power(r, n.suc), s)
} by {
    if is_transitive(s) and relation_subset(r, s) {
        define p(k: Nat) -> Bool {
            relation_subset(relation_power(r, k.suc), s)
        }
        relation_power_one(r)
        relation_subset(relation_power(r, Nat.zero.suc), s)
        p(Nat.zero)
        forall(k: Nat) {
            if p(k) {
                p(k)
                relation_subset(relation_power(r, k.suc), s)
                forall(x: T, y: T) {
                    if relation_power(r, k.suc.suc, x, y) {
                        relation_power_suc(r, k.suc)
                        relation_compose_has_witness(relation_power(r, k.suc), r, x, y)
                        let z: T satisfy {
                            relation_power(r, k.suc, x, z) and r(z, y)
                        }
                        relation_subset_step(relation_power(r, k.suc), s, x, z)
                        s(x, z)
                        relation_subset_step(r, s, z, y)
                        s(z, y)
                        transitive_step(s, x, z, y)
                        s(x, y)
                    }
                }
                relation_subset(relation_power(r, k.suc.suc), s)
                p(k.suc)
            }
        }
        p(n)
        relation_subset(relation_power(r, n.suc), s)
    }
}

/// Reflexive-transitive closure is the smallest reflexive transitive relation containing the base relation.
theorem relation_refl_trans_closure_subset_of_reflexive_transitive[T](r: (T, T) -> Bool, s: (T, T) -> Bool) {
    is_reflexive(s) and is_transitive(s) and relation_subset(r, s)
    implies
    relation_subset(relation_refl_trans_closure(r), s)
} by {
    if is_reflexive(s) and is_transitive(s) and relation_subset(r, s) {
        forall(x: T, y: T) {
            if relation_refl_trans_closure(r, x, y) {
                let n: Nat satisfy {
                    relation_power(r, n, x, y)
                }
                relation_power_subset_of_reflexive_transitive(r, s, n)
                relation_subset(relation_power(r, n), s)
                relation_subset_step(relation_power(r, n), s, x, y)
                s(x, y)
            }
        }
    }
}

/// Transitive closure is the smallest transitive relation containing the base relation.
theorem relation_trans_closure_subset_of_transitive[T](r: (T, T) -> Bool, s: (T, T) -> Bool) {
    is_transitive(s) and relation_subset(r, s)
    implies
    relation_subset(relation_trans_closure(r), s)
} by {
    if is_transitive(s) and relation_subset(r, s) {
        forall(x: T, y: T) {
            if relation_trans_closure(r, x, y) {
                let n: Nat satisfy {
                    relation_power(r, n.suc, x, y)
                }
                relation_power_positive_subset_of_transitive(r, s, n)
                relation_subset(relation_power(r, n.suc), s)
                relation_subset_step(relation_power(r, n.suc), s, x, y)
                s(x, y)
            }
        }
    }
}

/// Reflexive-transitive closure is transitive.
theorem relation_refl_trans_closure_is_transitive[T](r: (T, T) -> Bool) {
    is_transitive(relation_refl_trans_closure(r))
} by {
    forall(x: T, y: T, z: T) {
        if relation_refl_trans_closure(r, x, y) and relation_refl_trans_closure(r, y, z) {
            let m: Nat satisfy {
                relation_power(r, m, x, y)
            }
            let n: Nat satisfy {
                relation_power(r, n, y, z)
            }
            relation_compose_intro(relation_power(r, m), relation_power(r, n), x, y, z)
            relation_compose(relation_power(r, m), relation_power(r, n), x, z)
            relation_power_add(r, m, n)
            relation_power(r, m + n, x, z)
            relation_power_in_refl_trans_closure(r, m + n, x, z)
            relation_refl_trans_closure(r, x, z)
        }
    }
}

/// Transitive closure is transitive.
theorem relation_trans_closure_is_transitive[T](r: (T, T) -> Bool) {
    is_transitive(relation_trans_closure(r))
} by {
    forall(x: T, y: T, z: T) {
        if relation_trans_closure(r, x, y) and relation_trans_closure(r, y, z) {
            let m: Nat satisfy {
                relation_power(r, m.suc, x, y)
            }
            let n: Nat satisfy {
                relation_power(r, n.suc, y, z)
            }
            relation_compose_intro(relation_power(r, m.suc), relation_power(r, n.suc), x, y, z)
            relation_compose(relation_power(r, m.suc), relation_power(r, n.suc), x, z)
            relation_power_add(r, m.suc, n.suc)
            relation_power(r, m.suc + n.suc, x, z)
            add_suc_left(m, n.suc)
            relation_power_positive_in_trans_closure(r, m + n.suc, x, z)
            relation_trans_closure(r, x, z)
        }
    }
}

/// Transitive closure of a transitive relation equals itself.
theorem relation_trans_closure_eq_self_of_transitive[T](r: (T, T) -> Bool) {
    is_transitive(r) implies relation_trans_closure(r) = r
} by {
    if is_transitive(r) {
        relation_trans_closure_subset_of_transitive(r, r)
        relation_subset(relation_trans_closure(r), r)
        relation_subset_trans_closure(r)
        relation_subset(r, relation_trans_closure(r))
        relation_eq_iff_subset_both(relation_trans_closure(r), r)
        relation_trans_closure(r) = r
    }
}

/// Transitive closure is idempotent.
theorem relation_trans_closure_idempotent[T](r: (T, T) -> Bool) {
    relation_trans_closure(relation_trans_closure(r)) = relation_trans_closure(r)
} by {
    relation_trans_closure_is_transitive(r)
    relation_trans_closure_eq_self_of_transitive(relation_trans_closure(r))
}

/// Reflexive-transitive closure of a reflexive transitive relation equals itself.
theorem relation_refl_trans_closure_eq_self_of_reflexive_transitive[T](r: (T, T) -> Bool) {
    is_reflexive(r) and is_transitive(r) implies relation_refl_trans_closure(r) = r
} by {
    if is_reflexive(r) and is_transitive(r) {
        relation_refl_trans_closure_subset_of_reflexive_transitive(r, r)
        relation_subset(relation_refl_trans_closure(r), r)
        relation_subset_refl_trans_closure(r)
        relation_subset(r, relation_refl_trans_closure(r))
        relation_eq_iff_subset_both(relation_refl_trans_closure(r), r)
        relation_refl_trans_closure(r) = r
    }
}

/// Reflexive-transitive closure is idempotent.
theorem relation_refl_trans_closure_idempotent[T](r: (T, T) -> Bool) {
    relation_refl_trans_closure(relation_refl_trans_closure(r)) = relation_refl_trans_closure(r)
} by {
    relation_refl_trans_closure_is_reflexive(r)
    relation_refl_trans_closure_is_transitive(r)
    relation_refl_trans_closure_eq_self_of_reflexive_transitive(relation_refl_trans_closure(r))
}

/// Reflexive-transitive closure of a symmetric relation is symmetric.
theorem relation_refl_trans_closure_is_symmetric[T](r: (T, T) -> Bool) {
    is_symmetric(r) implies is_symmetric(relation_refl_trans_closure(r))
} by {
    if is_symmetric(r) {
        symmetric_imp_relation_converse_eq(r)
        relation_converse(r) = r
        forall(x: T, y: T) {
            if relation_refl_trans_closure(r, x, y) {
                relation_refl_trans_closure_converse(r, y, x)
                relation_refl_trans_closure(relation_converse(r), y, x) = relation_refl_trans_closure(r, x, y)
                relation_refl_trans_closure(relation_converse(r), y, x)
                relation_refl_trans_closure(r, y, x)
            }
        }
    }
}

/// The equivalence closure of a homogeneous relation: the smallest equivalence relation containing it.
define relation_equivalence_closure[T](r: (T, T) -> Bool, x: T, y: T) -> Bool {
    relation_refl_trans_closure(relation_symmetric_closure(r), x, y)
}

/// Every relation is contained in its equivalence closure.
theorem relation_subset_equivalence_closure[T](r: (T, T) -> Bool) {
    relation_subset(r, relation_equivalence_closure(r))
} by {
    relation_subset_symmetric_closure(r)
    relation_subset_refl_trans_closure(relation_symmetric_closure(r))
    forall(x: T, y: T) {
        if r(x, y) {
            relation_subset_step(r, relation_symmetric_closure(r), x, y)
            relation_symmetric_closure(r, x, y)
            relation_subset_step(relation_symmetric_closure(r), relation_refl_trans_closure(relation_symmetric_closure(r)), x, y)
            relation_refl_trans_closure(relation_symmetric_closure(r), x, y)
            relation_equivalence_closure(r, x, y)
        }
    }
}

/// Equivalence closure is reflexive.
theorem relation_equivalence_closure_is_reflexive[T](r: (T, T) -> Bool) {
    is_reflexive(relation_equivalence_closure(r))
} by {
    relation_refl_trans_closure_is_reflexive(relation_symmetric_closure(r))
    forall(x: T) {
        relation_refl_trans_closure(relation_symmetric_closure(r), x, x)
        relation_equivalence_closure(r, x, x)
    }
}

/// Equivalence closure is symmetric.
theorem relation_equivalence_closure_is_symmetric[T](r: (T, T) -> Bool) {
    is_symmetric(relation_equivalence_closure(r))
} by {
    relation_symmetric_closure_is_symmetric(r)
    relation_refl_trans_closure_is_symmetric(relation_symmetric_closure(r))
    forall(x: T, y: T) {
        if relation_equivalence_closure(r, x, y) {
            relation_refl_trans_closure(relation_symmetric_closure(r), x, y)
            relation_refl_trans_closure(relation_symmetric_closure(r), y, x)
            relation_equivalence_closure(r, y, x)
        }
    }
}

/// Equivalence closure is transitive.
theorem relation_equivalence_closure_is_transitive[T](r: (T, T) -> Bool) {
    is_transitive(relation_equivalence_closure(r))
} by {
    relation_refl_trans_closure_is_transitive(relation_symmetric_closure(r))
    forall(x: T, y: T, z: T) {
        if relation_equivalence_closure(r, x, y) and relation_equivalence_closure(r, y, z) {
            relation_refl_trans_closure(relation_symmetric_closure(r), x, y)
            relation_refl_trans_closure(relation_symmetric_closure(r), y, z)
            transitive_step(relation_refl_trans_closure(relation_symmetric_closure(r)), x, y, z)
            relation_refl_trans_closure(relation_symmetric_closure(r), x, z)
            relation_equivalence_closure(r, x, z)
        }
    }
}

/// Equivalence closure is an equivalence relation.
theorem relation_equivalence_closure_is_equivalence[T](r: (T, T) -> Bool) {
    is_equivalence(relation_equivalence_closure(r))
} by {
    relation_equivalence_closure_is_reflexive(r)
    relation_equivalence_closure_is_symmetric(r)
    relation_equivalence_closure_is_transitive(r)
}

/// Equivalence closure is the smallest equivalence relation containing the original relation.
theorem relation_equivalence_closure_subset_of_equivalence[T](r: (T, T) -> Bool, s: (T, T) -> Bool) {
    is_equivalence(s) and relation_subset(r, s) implies relation_subset(relation_equivalence_closure(r), s)
} by {
    if is_equivalence(s) and relation_subset(r, s) {
        is_equivalence(s) = is_reflexive(s) and is_symmetric(s) and is_transitive(s)
        relation_symmetric_closure_subset_of_symmetric(r, s)
        relation_subset(relation_symmetric_closure(r), s)
        relation_refl_trans_closure_subset_of_reflexive_transitive(relation_symmetric_closure(r), s)
        relation_subset(relation_refl_trans_closure(relation_symmetric_closure(r)), s)
        forall(x: T, y: T) {
            if relation_equivalence_closure(r, x, y) {
                relation_refl_trans_closure(relation_symmetric_closure(r), x, y)
                relation_subset_step(
                    relation_refl_trans_closure(relation_symmetric_closure(r)),
                    s,
                    x,
                    y
                )
                s(x, y)
            }
        }
    }
}

/// Equivalence closure is monotone in the base relation.
theorem relation_equivalence_closure_monotone[T](r: (T, T) -> Bool, s: (T, T) -> Bool) {
    relation_subset(r, s) implies
        relation_subset(relation_equivalence_closure(r), relation_equivalence_closure(s))
} by {
    if relation_subset(r, s) {
        relation_symmetric_closure_monotone(r, s)
        relation_subset(relation_symmetric_closure(r), relation_symmetric_closure(s))
        relation_refl_trans_closure_monotone(relation_symmetric_closure(r), relation_symmetric_closure(s))
        relation_subset(
            relation_refl_trans_closure(relation_symmetric_closure(r)),
            relation_refl_trans_closure(relation_symmetric_closure(s))
        )
        forall(x: T, y: T) {
            if relation_equivalence_closure(r, x, y) {
                relation_refl_trans_closure(relation_symmetric_closure(r), x, y)
                relation_subset_step(
                    relation_refl_trans_closure(relation_symmetric_closure(r)),
                    relation_refl_trans_closure(relation_symmetric_closure(s)),
                    x,
                    y
                )
                relation_refl_trans_closure(relation_symmetric_closure(s), x, y)
                relation_equivalence_closure(s, x, y)
            }
        }
    }
}

/// Equivalence closure of an equivalence relation equals itself.
theorem relation_equivalence_closure_eq_self_of_equivalence[T](r: (T, T) -> Bool) {
    is_equivalence(r) implies relation_equivalence_closure(r) = r
} by {
    if is_equivalence(r) {
        relation_subset(r, r)
        relation_equivalence_closure_subset_of_equivalence(r, r)
        relation_subset(relation_equivalence_closure(r), r)
        relation_subset_equivalence_closure(r)
        relation_subset(r, relation_equivalence_closure(r))
        relation_eq_iff_subset_both(relation_equivalence_closure(r), r)
        relation_equivalence_closure(r) = r
    }
}

/// Equivalence closure is idempotent.
theorem relation_equivalence_closure_idempotent[T](r: (T, T) -> Bool) {
    relation_equivalence_closure(relation_equivalence_closure(r)) = relation_equivalence_closure(r)
} by {
    relation_equivalence_closure_is_equivalence(r)
    relation_equivalence_closure_eq_self_of_equivalence(relation_equivalence_closure(r))
}

/// Prepending a single step extends the transitive closure on the left.
theorem relation_trans_closure_step_left[T](r: (T, T) -> Bool, x: T, y: T, z: T) {
    r(x, y) and relation_trans_closure(r, y, z) implies relation_trans_closure(r, x, z)
} by {
    if r(x, y) and relation_trans_closure(r, y, z) {
        relation_subset_trans_closure(r)
        relation_subset_step(r, relation_trans_closure(r), x, y)
        relation_trans_closure(r, x, y)
        relation_trans_closure_is_transitive(r)
        transitive_step(relation_trans_closure(r), x, y, z)
    }
}

/// Appending a single step extends the transitive closure on the right.
theorem relation_trans_closure_step_right[T](r: (T, T) -> Bool, x: T, y: T, z: T) {
    relation_trans_closure(r, x, y) and r(y, z) implies relation_trans_closure(r, x, z)
} by {
    if relation_trans_closure(r, x, y) and r(y, z) {
        relation_subset_trans_closure(r)
        relation_subset_step(r, relation_trans_closure(r), y, z)
        relation_trans_closure(r, y, z)
        relation_trans_closure_is_transitive(r)
        transitive_step(relation_trans_closure(r), x, y, z)
    }
}

/// Prepending a single step extends the reflexive-transitive closure on the left.
theorem relation_refl_trans_closure_step_left[T](r: (T, T) -> Bool, x: T, y: T, z: T) {
    r(x, y) and relation_refl_trans_closure(r, y, z) implies relation_refl_trans_closure(r, x, z)
} by {
    if r(x, y) and relation_refl_trans_closure(r, y, z) {
        relation_subset_refl_trans_closure(r)
        relation_subset_step(r, relation_refl_trans_closure(r), x, y)
        relation_refl_trans_closure(r, x, y)
        relation_refl_trans_closure_is_transitive(r)
        transitive_step(relation_refl_trans_closure(r), x, y, z)
    }
}

/// Appending a single step extends the reflexive-transitive closure on the right.
theorem relation_refl_trans_closure_step_right[T](r: (T, T) -> Bool, x: T, y: T, z: T) {
    relation_refl_trans_closure(r, x, y) and r(y, z) implies relation_refl_trans_closure(r, x, z)
} by {
    if relation_refl_trans_closure(r, x, y) and r(y, z) {
        relation_subset_refl_trans_closure(r)
        relation_subset_step(r, relation_refl_trans_closure(r), y, z)
        relation_refl_trans_closure(r, y, z)
        relation_refl_trans_closure_is_transitive(r)
        transitive_step(relation_refl_trans_closure(r), x, y, z)
    }
}

/// Composition of the transitive closure with itself stays in the transitive closure.
theorem relation_trans_closure_compose_subset[T](r: (T, T) -> Bool) {
    relation_subset(
        relation_compose(relation_trans_closure(r), relation_trans_closure(r)),
        relation_trans_closure(r)
    )
} by {
    relation_trans_closure_is_transitive(r)
    forall(x: T, z: T) {
        if relation_compose(relation_trans_closure(r), relation_trans_closure(r), x, z) {
            relation_compose_has_witness(relation_trans_closure(r), relation_trans_closure(r), x, z)
            let y: T satisfy {
                relation_trans_closure(r, x, y) and relation_trans_closure(r, y, z)
            }
            transitive_step(relation_trans_closure(r), x, y, z)
        }
    }
}

/// Composition of the reflexive-transitive closure with itself stays in the reflexive-transitive closure.
theorem relation_refl_trans_closure_compose_subset[T](r: (T, T) -> Bool) {
    relation_subset(
        relation_compose(relation_refl_trans_closure(r), relation_refl_trans_closure(r)),
        relation_refl_trans_closure(r)
    )
} by {
    relation_refl_trans_closure_is_transitive(r)
    forall(x: T, z: T) {
        if relation_compose(relation_refl_trans_closure(r), relation_refl_trans_closure(r), x, z) {
            relation_compose_has_witness(relation_refl_trans_closure(r), relation_refl_trans_closure(r), x, z)
            let y: T satisfy {
                relation_refl_trans_closure(r, x, y) and relation_refl_trans_closure(r, y, z)
            }
            transitive_step(relation_refl_trans_closure(r), x, y, z)
        }
    }
}

/// Equivalence closure commutes with converse.
theorem relation_equivalence_closure_converse[T](r: (T, T) -> Bool) {
    relation_equivalence_closure(relation_converse(r)) = relation_equivalence_closure(r)
} by {
    relation_equivalence_closure_is_equivalence(r)
    is_symmetric(relation_equivalence_closure(r))
    relation_subset_equivalence_closure(r)
    relation_subset(r, relation_equivalence_closure(r))
    forall(x: T, y: T) {
        if relation_converse(r, x, y) {
            r(y, x)
            relation_subset_step(r, relation_equivalence_closure(r), y, x)
            relation_equivalence_closure(r, y, x)
            is_symmetric(relation_equivalence_closure(r)) = forall(a: T, b: T) {
                relation_equivalence_closure(r, a, b) implies relation_equivalence_closure(r, b, a)
            }
            relation_equivalence_closure(r, x, y)
        }
    }
    relation_subset(relation_converse(r), relation_equivalence_closure(r))
    relation_equivalence_closure_subset_of_equivalence(relation_converse(r), relation_equivalence_closure(r))
    relation_subset(relation_equivalence_closure(relation_converse(r)), relation_equivalence_closure(r))

    relation_equivalence_closure_is_equivalence(relation_converse(r))
    is_symmetric(relation_equivalence_closure(relation_converse(r)))
    relation_subset_equivalence_closure(relation_converse(r))
    relation_subset(relation_converse(r), relation_equivalence_closure(relation_converse(r)))
    forall(x: T, y: T) {
        if r(x, y) {
            relation_converse(r, y, x)
            relation_subset_step(relation_converse(r), relation_equivalence_closure(relation_converse(r)), y, x)
            relation_equivalence_closure(relation_converse(r), y, x)
            is_symmetric(relation_equivalence_closure(relation_converse(r))) = forall(a: T, b: T) {
                relation_equivalence_closure(relation_converse(r), a, b) implies relation_equivalence_closure(relation_converse(r), b, a)
            }
            relation_equivalence_closure(relation_converse(r), x, y)
        }
    }
    relation_subset(r, relation_equivalence_closure(relation_converse(r)))
    relation_equivalence_closure_subset_of_equivalence(r, relation_equivalence_closure(relation_converse(r)))
    relation_subset(relation_equivalence_closure(r), relation_equivalence_closure(relation_converse(r)))

    relation_eq_iff_subset_both(relation_equivalence_closure(relation_converse(r)), relation_equivalence_closure(r))
}

/// One step then any number of steps lies in the transitive closure.
theorem relation_compose_subset_trans_closure_left[T](r: (T, T) -> Bool) {
    relation_subset(
        relation_compose(r, relation_refl_trans_closure(r)),
        relation_trans_closure(r)
    )
} by {
    forall(x: T, z: T) {
        if relation_compose(r, relation_refl_trans_closure(r), x, z) {
            relation_compose_has_witness(r, relation_refl_trans_closure(r), x, z)
            let y: T satisfy {
                r(x, y) and relation_refl_trans_closure(r, y, z)
            }
            let n: Nat satisfy {
                relation_power(r, n, y, z)
            }
            relation_compose_intro(r, relation_power(r, n), x, y, z)
            relation_compose(r, relation_power(r, n), x, z)
            relation_power_suc_left(r, n)
            relation_power(r, n.suc, x, z)
            relation_power_positive_in_trans_closure(r, n, x, z)
            relation_trans_closure(r, x, z)
        }
    }
}

/// Any number of steps then one step lies in the transitive closure.
theorem relation_compose_subset_trans_closure_right[T](r: (T, T) -> Bool) {
    relation_subset(
        relation_compose(relation_refl_trans_closure(r), r),
        relation_trans_closure(r)
    )
} by {
    forall(x: T, z: T) {
        if relation_compose(relation_refl_trans_closure(r), r, x, z) {
            relation_compose_has_witness(relation_refl_trans_closure(r), r, x, z)
            let y: T satisfy {
                relation_refl_trans_closure(r, x, y) and r(y, z)
            }
            let n: Nat satisfy {
                relation_power(r, n, x, y)
            }
            relation_compose_intro(relation_power(r, n), r, x, y, z)
            relation_compose(relation_power(r, n), r, x, z)
            relation_power_suc(r, n)
            relation_power(r, n.suc, x, z)
            relation_power_positive_in_trans_closure(r, n, x, z)
            relation_trans_closure(r, x, z)
        }
    }
}

/// Composition of the reflexive-transitive closure with itself equals the reflexive-transitive closure.
theorem relation_refl_trans_closure_compose_eq[T](r: (T, T) -> Bool) {
    relation_compose(relation_refl_trans_closure(r), relation_refl_trans_closure(r)) =
        relation_refl_trans_closure(r)
} by {
    let u = relation_compose(relation_refl_trans_closure(r), relation_refl_trans_closure(r))
    let v = relation_refl_trans_closure(r)
    relation_refl_trans_closure_compose_subset(r)
    relation_subset(u, v)
    relation_refl_trans_closure_is_reflexive(r)
    forall(x: T, y: T) {
        if v(x, y) {
            reflexive_self(v, y)
            v(y, y)
            relation_compose_intro(v, v, x, y, y)
            u(x, y)
        }
    }
    relation_subset(v, u)
    relation_eq_iff_subset_both(u, v)
}
