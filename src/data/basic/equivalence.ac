/// Generic infrastructure for equivalence classes.

from data.basic.relation_basic import is_equivalence, is_reflexive, is_symmetric, is_transitive
from data.basic.relation_transport import respects_unary_op, respects_binary_op, respects_unary_op_step,
    respects_binary_op_step, respects_function, respects_predicate,
    respects_predicate_eq_respects_function, is_unary_congruence, is_binary_congruence
from data.basic.functions import Inhabited, is_surjective_fn, surjective_fn_has_preimage
from data.basic.set import Set, equivalence_class, equivalence_class_eq_of_equiv,
    equivalence_of_equivalence_class_eq, set_ext, is_saturated_under

/// True if `c` is an equivalence class for the relation `r`.
define is_equivalence_class[T](r: (T, T) -> Bool, c: Set[T]) -> Bool {
    exists(a: T) {
        c = equivalence_class(r, a)
    }
}

/// The class of an element is an equivalence class.
theorem equivalence_class_is_equivalence_class[T](r: (T, T) -> Bool, a: T) {
    is_equivalence_class(r, equivalence_class(r, a))
} by {
}

/// An equivalence class has a representative.
theorem equivalence_class_has_representative[T](r: (T, T) -> Bool, c: Set[T]) {
    is_equivalence_class(r, c) implies exists(a: T) {
        c = equivalence_class(r, a)
    }
} by {
    if is_equivalence_class(r, c) {
        exists(a: T) {
            c = equivalence_class(r, a)
        }
    }
}

/// Equality of equivalence classes is equivalence of representatives.
theorem equivalence_class_eq_iff[T](r: (T, T) -> Bool, a: T, b: T) {
    is_equivalence(r) implies
    equivalence_class(r, a) = equivalence_class(r, b) = r(a, b)
} by {
    if is_equivalence(r) {
        if equivalence_class(r, a) = equivalence_class(r, b) {
            equivalence_of_equivalence_class_eq(r, a, b)
            r(a, b)
        }
        if r(a, b) {
            equivalence_class_eq_of_equiv(r, a, b)
            equivalence_class(r, a) = equivalence_class(r, b)
        }
        equivalence_class(r, a) = equivalence_class(r, b) = r(a, b)
    }
}

/// A relation together with the fact that it is an equivalence relation.
structure QuotientRelation[T] {
    /// The relation whose equivalence classes form quotient elements.
    rel: (T, T) -> Bool
} constraint {
    is_equivalence(rel)
}

/// Construction of a quotient relation remembers the underlying relation.
theorem quotient_relation_new_round_trip[T](r: (T, T) -> Bool, qrel: QuotientRelation[T]) {
    QuotientRelation[T].new(r) = Option.some(qrel) implies qrel.rel = r
}

/// Every quotient relation is reconstructed from its underlying relation.
theorem quotient_relation_new_self[T](qrel: QuotientRelation[T]) {
    QuotientRelation[T].new(qrel.rel) = Option.some(qrel)
}

/// Equality of options containing quotient relations is equality of those relations.
theorem quotient_relation_some_injective[T](qrel: QuotientRelation[T], srel: QuotientRelation[T]) {
    Option.some(qrel) = Option.some(srel) implies qrel = srel
}

/// Quotient relations are equal when their underlying relations are equal.
theorem quotient_relation_ext[T](qrel: QuotientRelation[T], srel: QuotientRelation[T]) {
    qrel.rel = srel.rel implies qrel = srel
} by {
    if qrel.rel = srel.rel {
        Option.some(qrel) = Option.some(srel)
        quotient_relation_some_injective(qrel, srel)
    }
}

/// Equal quotient relations have equal underlying relations.
theorem quotient_relation_eq_rel[T](qrel: QuotientRelation[T], srel: QuotientRelation[T]) {
    qrel = srel implies qrel.rel = srel.rel
}

/// Equality of quotient relations transports predicates on quotient relations.
theorem quotient_relation_eq_transport_predicate[T](
    p: QuotientRelation[T] -> Bool,
    qrel: QuotientRelation[T],
    srel: QuotientRelation[T]
) {
    qrel = srel and p(qrel) implies p(srel)
}

/// Equality of quotient relations transports predicates on quotient relations in the reverse direction.
theorem quotient_relation_eq_transport_predicate_rev[T](
    p: QuotientRelation[T] -> Bool,
    qrel: QuotientRelation[T],
    srel: QuotientRelation[T]
) {
    qrel = srel and p(srel) implies p(qrel)
}

/// Equality of underlying relations determines equality of quotient relations.
theorem quotient_relation_eq_of_rel_eq[T](qrel: QuotientRelation[T], srel: QuotientRelation[T]) {
    qrel.rel = srel.rel implies qrel = srel
} by {
    if qrel.rel = srel.rel {
        quotient_relation_ext(qrel, srel)
    }
}

/// A quotient element whose relation is carried as part of the element.
structure QuotientOver[T] {
    /// The equivalence relation whose classes form this quotient element.
    qrel: QuotientRelation[T]

    /// The equivalence class represented by this quotient element.
    carrier: Set[T]
} constraint {
    is_equivalence_class(qrel.rel, carrier)
}

/// The quotient-over element represented by `a`.
let quotient_over_mk[T](qrel: QuotientRelation[T], a: T) -> result: QuotientOver[T] satisfy {
    QuotientOver[T].new(qrel, equivalence_class(qrel.rel, a)) = Option.some(result)
} by {
    equivalence_class_is_equivalence_class(qrel.rel, a)
}

/// Construction of a quotient-over element remembers its relation.
theorem quotient_over_new_qrel[T](qrel: QuotientRelation[T], c: Set[T], q: QuotientOver[T]) {
    QuotientOver[T].new(qrel, c) = Option.some(q) implies q.qrel = qrel
}

/// Construction of a quotient-over element remembers its class.
theorem quotient_over_new_carrier[T](qrel: QuotientRelation[T], c: Set[T], q: QuotientOver[T]) {
    QuotientOver[T].new(qrel, c) = Option.some(q) implies q.carrier = c
}

/// Every quotient-over element is reconstructed from its relation and class.
theorem quotient_over_new_self[T](q: QuotientOver[T]) {
    QuotientOver[T].new(q.qrel, q.carrier) = Option.some(q)
}

/// Equality of options containing quotient-over elements is equality of those elements.
theorem quotient_over_some_injective[T](a: QuotientOver[T], b: QuotientOver[T]) {
    Option.some(a) = Option.some(b) implies a = b
}

/// Quotient-over elements are equal when their relations and classes are equal.
theorem quotient_over_ext[T](a: QuotientOver[T], b: QuotientOver[T]) {
    a.qrel = b.qrel and a.carrier = b.carrier implies a = b
} by {
    if a.qrel = b.qrel and a.carrier = b.carrier {
        Option.some(a) = Option.some(b)
        quotient_over_some_injective(a, b)
    }
}

/// Equal quotient-over elements have equal quotient relations.
theorem quotient_over_eq_qrel[T](a: QuotientOver[T], b: QuotientOver[T]) {
    a = b implies a.qrel = b.qrel
}

/// Equal quotient-over elements have equal carriers.
theorem quotient_over_eq_carrier[T](a: QuotientOver[T], b: QuotientOver[T]) {
    a = b implies a.carrier = b.carrier
}

/// Equality of quotient-over elements transports predicates on quotient-over elements.
theorem quotient_over_eq_transport_predicate[T](
    p: QuotientOver[T] -> Bool,
    a: QuotientOver[T],
    b: QuotientOver[T]
) {
    a = b and p(a) implies p(b)
}

/// Equality of quotient-over elements transports predicates on quotient-over elements in the reverse direction.
theorem quotient_over_eq_transport_predicate_rev[T](
    p: QuotientOver[T] -> Bool,
    a: QuotientOver[T],
    b: QuotientOver[T]
) {
    a = b and p(b) implies p(a)
}

/// Equality of fields determines equality of quotient-over elements.
theorem quotient_over_eq_of_fields_eq[T](a: QuotientOver[T], b: QuotientOver[T]) {
    a.qrel = b.qrel and a.carrier = b.carrier implies a = b
} by {
    if a.qrel = b.qrel and a.carrier = b.carrier {
        quotient_over_ext(a, b)
    }
}

/// The quotient-over projection remembers its relation and representative class.
theorem quotient_over_mk_fields[T](qrel: QuotientRelation[T], a: T) {
    quotient_over_mk(qrel, a).qrel = qrel and
    quotient_over_mk(qrel, a).carrier = equivalence_class(qrel.rel, a)
} by {
    QuotientOver[T].new(qrel, equivalence_class(qrel.rel, a)) =
        Option.some(quotient_over_mk(qrel, a))
    quotient_over_new_qrel(qrel, equivalence_class(qrel.rel, a), quotient_over_mk(qrel, a))
    quotient_over_new_carrier(qrel, equivalence_class(qrel.rel, a), quotient_over_mk(qrel, a))
}

/// A quotient-over element is the projection of any representative of its class.
theorem quotient_over_eq_mk_of_carrier[T](qrel: QuotientRelation[T], q: QuotientOver[T], a: T) {
    q.qrel = qrel and q.carrier = equivalence_class(qrel.rel, a) implies
    quotient_over_mk(qrel, a) = q
} by {
    if q.qrel = qrel and q.carrier = equivalence_class(qrel.rel, a) {
        quotient_over_some_injective(q, quotient_over_mk(qrel, a))
        quotient_over_mk(qrel, a) = q
    }
}

/// Every quotient-over element has a representative.
theorem quotient_over_mk_surjective[T](q: QuotientOver[T]) {
    exists(a: T) {
        quotient_over_mk(q.qrel, a) = q
    }
} by {
    equivalence_class_has_representative(q.qrel.rel, q.carrier)
    let a: T satisfy {
        q.carrier = equivalence_class(q.qrel.rel, a)
    }
    quotient_over_eq_mk_of_carrier(q.qrel, q, a)
}

/// Related representatives determine the same quotient-over element.
theorem quotient_over_mk_eq_of_rel[T](qrel: QuotientRelation[T], a: T, b: T) {
    qrel.rel(a, b) implies quotient_over_mk(qrel, a) = quotient_over_mk(qrel, b)
} by {
    if qrel.rel(a, b) {
        equivalence_class_eq_of_equiv(qrel.rel, a, b)
        QuotientOver[T].new(qrel, equivalence_class(qrel.rel, a)) =
            QuotientOver[T].new(qrel, equivalence_class(qrel.rel, b))
        quotient_over_some_injective(quotient_over_mk(qrel, a), quotient_over_mk(qrel, b))
        quotient_over_mk(qrel, a) = quotient_over_mk(qrel, b)
    }
}

/// Equality of quotient-over representatives forces the representatives to be related.
theorem rel_of_quotient_over_mk_eq[T](qrel: QuotientRelation[T], a: T, b: T) {
    quotient_over_mk(qrel, a) = quotient_over_mk(qrel, b) implies qrel.rel(a, b)
} by {
    if quotient_over_mk(qrel, a) = quotient_over_mk(qrel, b) {
        quotient_over_mk_fields(qrel, a)
        quotient_over_mk_fields(qrel, b)
        equivalence_of_equivalence_class_eq(qrel.rel, a, b)
        qrel.rel(a, b)
    }
}

/// Equality of quotient-over representatives is the underlying equivalence relation.
theorem quotient_over_mk_eq_iff_rel[T](qrel: QuotientRelation[T], a: T, b: T) {
    (quotient_over_mk(qrel, a) = quotient_over_mk(qrel, b)) = qrel.rel(a, b)
} by {
    if quotient_over_mk(qrel, a) = quotient_over_mk(qrel, b) {
        rel_of_quotient_over_mk_eq(qrel, a, b)
        qrel.rel(a, b)
    }
    if qrel.rel(a, b) {
        quotient_over_mk_eq_of_rel(qrel, a, b)
        quotient_over_mk(qrel, a) = quotient_over_mk(qrel, b)
    }
}

/// True if a function carries related representatives to related representatives.
define quotient_over_respects[T, U](
    qrel: QuotientRelation[T],
    srel: QuotientRelation[U],
    f: T -> U
) -> Bool {
    forall(a: T, b: T) {
        qrel.rel(a, b) implies srel.rel(f(a), f(b))
    }
}

/// The quotient-over value obtained by lifting a representative function.
let quotient_over_lift_at[T, U](srel: QuotientRelation[U], f: T -> U, q: QuotientOver[T]) -> result: QuotientOver[U] satisfy {
    exists(a: T) {
        quotient_over_mk(q.qrel, a) = q and result = quotient_over_mk(srel, f(a))
    }
} by {
    quotient_over_mk_surjective(q)
    let a: T satisfy {
        quotient_over_mk(q.qrel, a) = q
    }
}

/// The quotient-over lift has a representative description.
theorem quotient_over_lift_at_has_representative[T, U](
    srel: QuotientRelation[U],
    f: T -> U,
    q: QuotientOver[T]
) {
    exists(a: T) {
        quotient_over_mk(q.qrel, a) = q and quotient_over_lift_at(srel, f, q) =
            quotient_over_mk(srel, f(a))
    }
}

/// The quotient-over lift agrees with the representative function on projections.
theorem quotient_over_lift_at_projection[T, U](
    qrel: QuotientRelation[T],
    srel: QuotientRelation[U],
    f: T -> U,
    a: T
) {
    quotient_over_respects(qrel, srel, f) implies
    quotient_over_lift_at(srel, f, quotient_over_mk(qrel, a)) = quotient_over_mk(srel, f(a))
} by {
    if quotient_over_respects(qrel, srel, f) {
        quotient_over_lift_at_has_representative(srel, f, quotient_over_mk(qrel, a))
        let b: T satisfy {
            quotient_over_mk(qrel, b) = quotient_over_mk(qrel, a) and
            quotient_over_lift_at(srel, f, quotient_over_mk(qrel, a)) = quotient_over_mk(srel, f(b))
        }
        rel_of_quotient_over_mk_eq(qrel, b, a)
        quotient_over_respects(qrel, srel, f) = forall(x: T, y: T) {
            qrel.rel(x, y) implies srel.rel(f(x), f(y))
        }
        quotient_over_mk_eq_of_rel(srel, f(b), f(a))
        quotient_over_mk(srel, f(b)) = quotient_over_mk(srel, f(a))
        quotient_over_lift_at(srel, f, quotient_over_mk(qrel, a)) = quotient_over_mk(srel, f(a))
    }
}

/// The function on quotient-over elements induced by a representative function.
define quotient_over_lift[T, U](srel: QuotientRelation[U], f: T -> U) -> QuotientOver[T] -> QuotientOver[U] {
    function(q: QuotientOver[T]) {
        quotient_over_lift_at(srel, f, q)
    }
}

/// The quotient-over lift agrees with the representative function on projections.
theorem quotient_over_lift_projection[T, U](
    qrel: QuotientRelation[T],
    srel: QuotientRelation[U],
    f: T -> U,
    a: T
) {
    quotient_over_respects(qrel, srel, f) implies
    quotient_over_lift(srel, f, quotient_over_mk(qrel, a)) = quotient_over_mk(srel, f(a))
} by {
    if quotient_over_respects(qrel, srel, f) {
        quotient_over_lift(srel, f, quotient_over_mk(qrel, a)) =
            quotient_over_lift_at(srel, f, quotient_over_mk(qrel, a))
        quotient_over_lift_at_projection(qrel, srel, f, a)
    }
}

/// The quotient of a type by a relation, represented by equivalence classes.
structure Quotient[T] {
    /// The equivalence relation whose classes form this quotient.
    rel: (T, T) -> Bool

    /// The equivalence class represented by this quotient element.
    carrier: Set[T]
} constraint {
    is_equivalence(rel) and is_equivalence_class(rel, carrier)
}

/// The quotient element represented by `a`.
let quotient_mk[T](qrel: QuotientRelation[T], a: T) -> result: Quotient[T] satisfy {
    Quotient[T].new(qrel.rel, equivalence_class(qrel.rel, a)) = Option.some(result)
} by {
    is_equivalence(qrel.rel)
    equivalence_class_is_equivalence_class(qrel.rel, a)
    is_equivalence_class(qrel.rel, equivalence_class(qrel.rel, a))
}

/// Construction of a quotient element remembers its relation.
theorem quotient_new_rel[T](r: (T, T) -> Bool, c: Set[T], q: Quotient[T]) {
    Quotient[T].new(r, c) = Option.some(q) implies q.rel = r
}

/// Construction of a quotient element remembers its class.
theorem quotient_new_carrier[T](r: (T, T) -> Bool, c: Set[T], q: Quotient[T]) {
    Quotient[T].new(r, c) = Option.some(q) implies q.carrier = c
}

/// Every quotient element is reconstructed from its relation and class.
theorem quotient_new_self[T](q: Quotient[T]) {
    Quotient[T].new(q.rel, q.carrier) = Option.some(q)
}

/// Equality of options containing quotient elements is equality of those elements.
theorem quotient_some_injective[T](a: Quotient[T], b: Quotient[T]) {
    Option.some(a) = Option.some(b) implies a = b
}

/// Quotient elements are equal when their relations and classes are equal.
theorem quotient_ext[T](a: Quotient[T], b: Quotient[T]) {
    a.rel = b.rel and a.carrier = b.carrier implies a = b
} by {
    if a.rel = b.rel and a.carrier = b.carrier {
        Option.some(a) = Option.some(b)
        quotient_some_injective(a, b)
    }
}

/// Equal quotient elements have equal underlying relations.
theorem quotient_eq_rel[T](a: Quotient[T], b: Quotient[T]) {
    a = b implies a.rel = b.rel
}

/// Equal quotient elements have equal carriers.
theorem quotient_eq_carrier[T](a: Quotient[T], b: Quotient[T]) {
    a = b implies a.carrier = b.carrier
}

/// Equality of quotient elements transports predicates on quotient elements.
theorem quotient_eq_transport_predicate[T](p: Quotient[T] -> Bool, a: Quotient[T], b: Quotient[T]) {
    a = b and p(a) implies p(b)
}

/// Equality of quotient elements transports predicates on quotient elements in the reverse direction.
theorem quotient_eq_transport_predicate_rev[T](p: Quotient[T] -> Bool, a: Quotient[T], b: Quotient[T]) {
    a = b and p(b) implies p(a)
}

/// Equality of fields determines equality of quotient elements.
theorem quotient_eq_of_fields_eq[T](a: Quotient[T], b: Quotient[T]) {
    a.rel = b.rel and a.carrier = b.carrier implies a = b
} by {
    if a.rel = b.rel and a.carrier = b.carrier {
        quotient_ext(a, b)
    }
}

/// The unconstrained quotient element associated to a quotient-over element.
let quotient_over_to_quotient[T](q: QuotientOver[T]) -> result: Quotient[T] satisfy {
    Quotient[T].new(q.qrel.rel, q.carrier) = Option.some(result)
} by {
    is_equivalence(q.qrel.rel)
}

/// Conversion from quotient-over elements preserves the stored relation.
theorem quotient_over_to_quotient_rel[T](q: QuotientOver[T]) {
    quotient_over_to_quotient(q).rel = q.qrel.rel
} by {
    quotient_new_rel(q.qrel.rel, q.carrier, quotient_over_to_quotient(q))
}

/// Conversion from quotient-over elements preserves the class.
theorem quotient_over_to_quotient_carrier[T](q: QuotientOver[T]) {
    quotient_over_to_quotient(q).carrier = q.carrier
} by {
    quotient_new_carrier(q.qrel.rel, q.carrier, quotient_over_to_quotient(q))
}

/// A quotient with a matching relation has an associated quotient-over element.
theorem quotient_to_quotient_over_exists[T](qrel: QuotientRelation[T], q: Quotient[T]) {
    q.rel = qrel.rel implies exists(qo: QuotientOver[T]) {
        qo.qrel = qrel and qo.carrier = q.carrier
    }
} by {
    if q.rel = qrel.rel {
        is_equivalence_class(q.rel, q.carrier)
        is_equivalence_class(qrel.rel, q.carrier)
        let qo: QuotientOver[T] satisfy {
            QuotientOver[T].new(qrel, q.carrier) = Option.some(qo)
        }
        quotient_over_new_qrel(qrel, q.carrier, qo)
        quotient_over_new_carrier(qrel, q.carrier, qo)
        exists(result: QuotientOver[T]) {
            result.qrel = qrel and result.carrier = q.carrier
        }
    }
}

/// The quotient projection is built from the equivalence class of its representative.
theorem quotient_mk_new_self[T](qrel: QuotientRelation[T], a: T) {
    Quotient[T].new(qrel.rel, equivalence_class(qrel.rel, a)) = Option.some(quotient_mk(qrel, a))
}

/// The quotient projection remembers its relation and representative class.
theorem quotient_mk_fields[T](qrel: QuotientRelation[T], a: T) {
    quotient_mk(qrel, a).rel = qrel.rel and
    quotient_mk(qrel, a).carrier = equivalence_class(qrel.rel, a)
} by {
    quotient_mk_new_self(qrel, a)
    quotient_new_rel(qrel.rel, equivalence_class(qrel.rel, a), quotient_mk(qrel, a))
    quotient_new_carrier(qrel.rel, equivalence_class(qrel.rel, a), quotient_mk(qrel, a))
}

/// A quotient element is the projection of any representative of its class.
theorem quotient_eq_mk_of_carrier[T](qrel: QuotientRelation[T], q: Quotient[T], a: T) {
    q.rel = qrel.rel and q.carrier = equivalence_class(qrel.rel, a) implies
    quotient_mk(qrel, a) = q
} by {
    if q.rel = qrel.rel and q.carrier = equivalence_class(qrel.rel, a) {
        Option.some(q) = Option.some(quotient_mk(qrel, a))
        quotient_some_injective(q, quotient_mk(qrel, a))
        quotient_mk(qrel, a) = q
    }
}

/// Every quotient element for a relation has a representative.
theorem quotient_mk_surjective[T](qrel: QuotientRelation[T], q: Quotient[T]) {
    q.rel = qrel.rel implies exists(a: T) {
        quotient_mk(qrel, a) = q
    }
} by {
    if q.rel = qrel.rel {
        equivalence_class_has_representative(q.rel, q.carrier)
        let a: T satisfy {
            q.carrier = equivalence_class(q.rel, a)
        }
        quotient_eq_mk_of_carrier(qrel, q, a)
        exists(a0: T) {
            quotient_mk(qrel, a0) = q
        }
    }
}

/// A predicate on a quotient holds when it holds on all representatives.
theorem quotient_induction[T](qrel: QuotientRelation[T], p: Quotient[T] -> Bool, q: Quotient[T]) {
    q.rel = qrel.rel and forall(a: T) {
        p(quotient_mk(qrel, a))
    } implies p(q)
} by {
    if q.rel = qrel.rel and forall(a: T) {
        p(quotient_mk(qrel, a))
    } {
        quotient_mk_surjective(qrel, q)
        let a: T satisfy {
            quotient_mk(qrel, a) = q
        }
        p(q)
    }
}

/// Two quotient functions agree on a quotient element when they agree on representatives.
theorem quotient_function_eq_at[T, U](
    qrel: QuotientRelation[T],
    f: Quotient[T] -> U,
    g: Quotient[T] -> U,
    q: Quotient[T]
) {
    q.rel = qrel.rel and
    (forall(a: T) { f(quotient_mk(qrel, a)) = g(quotient_mk(qrel, a)) })
    implies f(q) = g(q)
} by {
    if q.rel = qrel.rel and
        (forall(a: T) { f(quotient_mk(qrel, a)) = g(quotient_mk(qrel, a)) }) {
        quotient_mk_surjective(qrel, q)
        let a: T satisfy {
            quotient_mk(qrel, a) = q
        }
        f(quotient_mk(qrel, a)) = g(quotient_mk(qrel, a))
        f(q) = g(q)
    }
}

/// Related representatives determine the same quotient element.
theorem quotient_mk_eq_of_rel[T](qrel: QuotientRelation[T], a: T, b: T) {
    qrel.rel(a, b) implies quotient_mk(qrel, a) = quotient_mk(qrel, b)
} by {
    if qrel.rel(a, b) {
        equivalence_class_eq_of_equiv(qrel.rel, a, b)
        Quotient[T].new(qrel.rel, equivalence_class(qrel.rel, a)) =
            Quotient[T].new(qrel.rel, equivalence_class(qrel.rel, b))
        quotient_some_injective(quotient_mk(qrel, a), quotient_mk(qrel, b))
        quotient_mk(qrel, a) = quotient_mk(qrel, b)
    }
}

/// Equality of projected representatives forces the representatives to be related.
theorem rel_of_quotient_mk_eq[T](qrel: QuotientRelation[T], a: T, b: T) {
    quotient_mk(qrel, a) = quotient_mk(qrel, b) implies qrel.rel(a, b)
} by {
    if quotient_mk(qrel, a) = quotient_mk(qrel, b) {
        quotient_mk_fields(qrel, a)
        quotient_mk_fields(qrel, b)
        equivalence_of_equivalence_class_eq(qrel.rel, a, b)
        qrel.rel(a, b)
    }
}

/// Equality of projected representatives is the underlying equivalence relation.
theorem quotient_mk_eq_iff_rel[T](qrel: QuotientRelation[T], a: T, b: T) {
    (quotient_mk(qrel, a) = quotient_mk(qrel, b)) = qrel.rel(a, b)
} by {
    if quotient_mk(qrel, a) = quotient_mk(qrel, b) {
        rel_of_quotient_mk_eq(qrel, a, b)
        qrel.rel(a, b)
    }
    if qrel.rel(a, b) {
        quotient_mk_eq_of_rel(qrel, a, b)
        quotient_mk(qrel, a) = quotient_mk(qrel, b)
    }
}

/// A relation-respecting function is constant on quotient representatives.
theorem quotient_lift_well_defined[T, U](qrel: QuotientRelation[T], f: T -> U, a: T, b: T) {
    (forall(x: T, y: T) { qrel.rel(x, y) implies f(x) = f(y) }) and
    quotient_mk(qrel, a) = quotient_mk(qrel, b) implies f(a) = f(b)
} by {
    if (forall(x: T, y: T) { qrel.rel(x, y) implies f(x) = f(y) }) and
        quotient_mk(qrel, a) = quotient_mk(qrel, b) {
        rel_of_quotient_mk_eq(qrel, a, b)
        f(a) = f(b)
    }
}

/// A quotient predicate is well-defined when it is invariant under the relation.
theorem quotient_predicate_well_defined[T](qrel: QuotientRelation[T], p: T -> Bool, a: T, b: T) {
    (forall(x: T, y: T) { qrel.rel(x, y) implies p(x) = p(y) }) and
    quotient_mk(qrel, a) = quotient_mk(qrel, b) implies p(a) = p(b)
} by {
    if (forall(x: T, y: T) { qrel.rel(x, y) implies p(x) = p(y) }) and
        quotient_mk(qrel, a) = quotient_mk(qrel, b) {
        quotient_lift_well_defined(qrel, p, a, b)
        p(a) = p(b)
    }
}

/// The projection from representatives to the quotient.
define quotient_projection[T](qrel: QuotientRelation[T], a: T) -> Quotient[T] {
    quotient_mk(qrel, a)
}

/// The quotient projection is the quotient constructor.
theorem quotient_projection_eq_mk[T](qrel: QuotientRelation[T], a: T) {
    quotient_projection(qrel, a) = quotient_mk(qrel, a)
}

/// Equality of projected representatives is the underlying equivalence relation.
theorem quotient_projection_eq_iff_rel[T](qrel: QuotientRelation[T], a: T, b: T) {
    (quotient_projection(qrel, a) = quotient_projection(qrel, b)) = qrel.rel(a, b)
} by {
    quotient_projection_eq_mk(qrel, a)
    quotient_projection_eq_mk(qrel, b)
    quotient_mk_eq_iff_rel(qrel, a, b)
}

/// Related representatives have the same quotient projection.
theorem quotient_projection_eq_of_rel[T](qrel: QuotientRelation[T], a: T, b: T) {
    qrel.rel(a, b) implies quotient_projection(qrel, a) = quotient_projection(qrel, b)
} by {
    if qrel.rel(a, b) {
        quotient_projection_eq_iff_rel(qrel, a, b)
        quotient_projection(qrel, a) = quotient_projection(qrel, b)
    }
}

/// Equal quotient projections force related representatives.
theorem rel_of_quotient_projection_eq[T](qrel: QuotientRelation[T], a: T, b: T) {
    quotient_projection(qrel, a) = quotient_projection(qrel, b) implies qrel.rel(a, b)
} by {
    if quotient_projection(qrel, a) = quotient_projection(qrel, b) {
        quotient_projection_eq_iff_rel(qrel, a, b)
        qrel.rel(a, b)
    }
}

/// Every quotient element over the relation is reached by the projection.
theorem quotient_projection_surjective_on_relation[T](qrel: QuotientRelation[T], q: Quotient[T]) {
    q.rel = qrel.rel implies exists(a: T) {
        quotient_projection(qrel, a) = q
    }
} by {
    if q.rel = qrel.rel {
        quotient_mk_surjective(qrel, q)
        let a: T satisfy {
            quotient_mk(qrel, a) = q
        }
        exists(a0: T) {
            quotient_projection(qrel, a0) = q
        }
    }
}

/// True if a representative function is invariant under a quotient relation.
define quotient_respects[T, U](qrel: QuotientRelation[T], f: T -> U) -> Bool {
    forall(a: T, b: T) {
        qrel.rel(a, b) implies f(a) = f(b)
    }
}

/// Quotient function invariance is relation-level function respect.
theorem quotient_respects_eq_respects_function[T, U](qrel: QuotientRelation[T], f: T -> U) {
    quotient_respects(qrel, f) = respects_function(qrel.rel, f)
} by {
    if quotient_respects(qrel, f) {
        quotient_respects(qrel, f) = forall(x: T, y: T) {
            qrel.rel(x, y) implies f(x) = f(y)
        }
        forall(x: T, y: T) {
            if qrel.rel(x, y) {
                f(x) = f(y)
            }
        }
        respects_function(qrel.rel, f)
    }
    if respects_function(qrel.rel, f) {
        respects_function(qrel.rel, f) = forall(x: T, y: T) {
            qrel.rel(x, y) implies f(x) = f(y)
        }
        forall(x: T, y: T) {
            if qrel.rel(x, y) {
                f(x) = f(y)
            }
        }
        quotient_respects(qrel, f)
    }
}

/// A quotient-respecting function has equal values on related representatives.
theorem quotient_respects_step[T, U](qrel: QuotientRelation[T], f: T -> U, a: T, b: T) {
    quotient_respects(qrel, f) and qrel.rel(a, b) implies f(a) = f(b)
} by {
    if quotient_respects(qrel, f) and qrel.rel(a, b) {
        quotient_respects(qrel, f) = forall(x: T, y: T) {
            qrel.rel(x, y) implies f(x) = f(y)
        }
        f(a) = f(b)
    }
}

/// Quotient predicate invariance is relation-level predicate respect.
theorem quotient_predicate_respects_eq[T](qrel: QuotientRelation[T], p: T -> Bool) {
    quotient_respects(qrel, p) = respects_predicate(qrel.rel, p)
} by {
    quotient_respects_eq_respects_function(qrel, p)
    respects_predicate_eq_respects_function(qrel.rel, p)
}

/// A quotient-respecting predicate transports truth forward along the relation.
theorem quotient_predicate_forward[T](qrel: QuotientRelation[T], p: T -> Bool, a: T, b: T) {
    quotient_respects(qrel, p) and qrel.rel(a, b) and p(a) implies p(b)
} by {
    if quotient_respects(qrel, p) and qrel.rel(a, b) and p(a) {
        quotient_respects_step(qrel, p, a, b)
        p(b)
    }
}

/// A quotient-respecting predicate transports truth backward along the relation.
theorem quotient_predicate_backward[T](qrel: QuotientRelation[T], p: T -> Bool, a: T, b: T) {
    quotient_respects(qrel, p) and qrel.rel(a, b) and p(b) implies p(a)
} by {
    if quotient_respects(qrel, p) and qrel.rel(a, b) and p(b) {
        quotient_respects_step(qrel, p, a, b)
        p(a)
    }
}

/// True if a unary operation preserves a quotient relation.
define quotient_unary_respects[T](qrel: QuotientRelation[T], f: T -> T) -> Bool {
    forall(a: T, b: T) {
        qrel.rel(a, b) implies qrel.rel(f(a), f(b))
    }
}

/// True if a binary operation preserves a quotient relation in both arguments.
define quotient_binary_respects[T](qrel: QuotientRelation[T], op: (T, T) -> T) -> Bool {
    forall(a1: T, a2: T, b1: T, b2: T) {
        qrel.rel(a1, a2) and qrel.rel(b1, b2) implies
        qrel.rel(op(a1, b1), op(a2, b2))
    }
}

/// Quotient unary compatibility is relation unary compatibility.
theorem quotient_unary_respects_eq_respects_unary_op[T](qrel: QuotientRelation[T], f: T -> T) {
    quotient_unary_respects(qrel, f) = respects_unary_op(qrel.rel, f)
} by {
    if quotient_unary_respects(qrel, f) {
        forall(a: T, b: T) {
            if qrel.rel(a, b) {
                quotient_unary_respects(qrel, f) = forall(x: T, y: T) {
                    qrel.rel(x, y) implies qrel.rel(f(x), f(y))
                }
                qrel.rel(f(a), f(b))
            }
        }
        respects_unary_op(qrel.rel, f)
    }
    if respects_unary_op(qrel.rel, f) {
        forall(a: T, b: T) {
            if qrel.rel(a, b) {
                respects_unary_op_step(qrel.rel, f, a, b)
                qrel.rel(f(a), f(b))
            }
        }
        quotient_unary_respects(qrel, f)
    }
}

/// Quotient binary compatibility is relation binary compatibility.
theorem quotient_binary_respects_eq_respects_binary_op[T](qrel: QuotientRelation[T], op: (T, T) -> T) {
    quotient_binary_respects(qrel, op) = respects_binary_op(qrel.rel, op)
} by {
    if quotient_binary_respects(qrel, op) {
        forall(a1: T, a2: T, b1: T, b2: T) {
            if qrel.rel(a1, a2) and qrel.rel(b1, b2) {
                quotient_binary_respects(qrel, op) = forall(x1: T, x2: T, y1: T, y2: T) {
                    qrel.rel(x1, x2) and qrel.rel(y1, y2) implies
                    qrel.rel(op(x1, y1), op(x2, y2))
                }
                qrel.rel(op(a1, b1), op(a2, b2))
            }
        }
        respects_binary_op(qrel.rel, op)
    }
    if respects_binary_op(qrel.rel, op) {
        forall(a1: T, a2: T, b1: T, b2: T) {
            if qrel.rel(a1, a2) and qrel.rel(b1, b2) {
                respects_binary_op_step(qrel.rel, op, a1, a2, b1, b2)
                qrel.rel(op(a1, b1), op(a2, b2))
            }
        }
        quotient_binary_respects(qrel, op)
    }
}

/// A quotient unary-compatible relation is a unary congruence.
theorem quotient_unary_respects_is_unary_congruence[T](qrel: QuotientRelation[T], f: T -> T) {
    quotient_unary_respects(qrel, f) implies is_unary_congruence(qrel.rel, f)
} by {
    if quotient_unary_respects(qrel, f) {
        quotient_unary_respects_eq_respects_unary_op(qrel, f)
        is_unary_congruence(qrel.rel, f)
    }
}

/// A quotient binary-compatible relation is a binary congruence.
theorem quotient_binary_respects_is_binary_congruence[T](qrel: QuotientRelation[T], op: (T, T) -> T) {
    quotient_binary_respects(qrel, op) implies is_binary_congruence(qrel.rel, op)
} by {
    if quotient_binary_respects(qrel, op) {
        quotient_binary_respects_eq_respects_binary_op(qrel, op)
        is_binary_congruence(qrel.rel, op)
    }
}

/// A relation-preserving unary operation gives equal quotient projections.
theorem quotient_unary_projection_compatible[T](qrel: QuotientRelation[T], f: T -> T, a: T, b: T) {
    quotient_unary_respects(qrel, f) and qrel.rel(a, b) implies
    quotient_projection(qrel, f(a)) = quotient_projection(qrel, f(b))
} by {
    if quotient_unary_respects(qrel, f) and qrel.rel(a, b) {
        quotient_unary_respects(qrel, f) = forall(x: T, y: T) {
            qrel.rel(x, y) implies qrel.rel(f(x), f(y))
        }
        qrel.rel(f(a), f(b))
        quotient_projection_eq_of_rel(qrel, f(a), f(b))
    }
}

/// A relation-preserving binary operation gives equal quotient projections.
theorem quotient_binary_projection_compatible[T](
    qrel: QuotientRelation[T],
    op: (T, T) -> T,
    a1: T,
    a2: T,
    b1: T,
    b2: T
) {
    quotient_binary_respects(qrel, op) and qrel.rel(a1, a2) and qrel.rel(b1, b2) implies
    quotient_projection(qrel, op(a1, b1)) = quotient_projection(qrel, op(a2, b2))
} by {
    if quotient_binary_respects(qrel, op) and qrel.rel(a1, a2) and qrel.rel(b1, b2) {
        quotient_binary_respects(qrel, op) = forall(x1: T, x2: T, y1: T, y2: T) {
            qrel.rel(x1, x2) and qrel.rel(y1, y2) implies
            qrel.rel(op(x1, y1), op(x2, y2))
        }
        qrel.rel(op(a1, b1), op(a2, b2))
        quotient_projection_eq_of_rel(qrel, op(a1, b1), op(a2, b2))
    }
}

/// A relation-preserving binary operation gives equal projections when the left input changes.
theorem quotient_binary_projection_compatible_left[T](
    qrel: QuotientRelation[T],
    op: (T, T) -> T,
    a1: T,
    a2: T,
    b: T
) {
    quotient_binary_respects(qrel, op) and qrel.rel(a1, a2) implies
    quotient_projection(qrel, op(a1, b)) = quotient_projection(qrel, op(a2, b))
} by {
    if quotient_binary_respects(qrel, op) and qrel.rel(a1, a2) {
        qrel.rel(b, b)
        quotient_binary_projection_compatible(qrel, op, a1, a2, b, b)
    }
}

/// A relation-preserving binary operation gives equal projections when the right input changes.
theorem quotient_binary_projection_compatible_right[T](
    qrel: QuotientRelation[T],
    op: (T, T) -> T,
    a: T,
    b1: T,
    b2: T
) {
    quotient_binary_respects(qrel, op) and qrel.rel(b1, b2) implies
    quotient_projection(qrel, op(a, b1)) = quotient_projection(qrel, op(a, b2))
} by {
    if quotient_binary_respects(qrel, op) and qrel.rel(b1, b2) {
        qrel.rel(a, a)
        quotient_binary_projection_compatible(qrel, op, a, a, b1, b2)
    }
}

/// The quotient-over unary operation induced by a representative operation.
define quotient_over_unary_op[T](qrel: QuotientRelation[T], f: T -> T) -> QuotientOver[T] -> QuotientOver[T] {
    function(q: QuotientOver[T]) {
        quotient_over_lift(qrel, f, q)
    }
}

/// The quotient-over constant induced by a representative.
define quotient_over_const[T](qrel: QuotientRelation[T], a: T) -> QuotientOver[T] {
    quotient_over_mk(qrel, a)
}

/// The quotient-over constant is the projection of its representative.
theorem quotient_over_const_projection[T](qrel: QuotientRelation[T], a: T) {
    quotient_over_const(qrel, a) = quotient_over_mk(qrel, a)
}

/// The quotient-over unary operation agrees with the representative operation on projections.
theorem quotient_over_unary_op_projection[T](qrel: QuotientRelation[T], f: T -> T, a: T) {
    quotient_unary_respects(qrel, f) implies
    quotient_over_unary_op(qrel, f, quotient_over_mk(qrel, a)) = quotient_over_mk(qrel, f(a))
} by {
    if quotient_unary_respects(qrel, f) {
        quotient_over_lift_projection(qrel, qrel, f, a)
        quotient_over_unary_op(qrel, f, quotient_over_mk(qrel, a)) =
            quotient_over_lift(qrel, f, quotient_over_mk(qrel, a))
    }
}

/// A relation-preserving unary operation on quotient-over elements respects related representatives.
theorem quotient_over_unary_op_projection_compatible[T](qrel: QuotientRelation[T], f: T -> T, a: T, b: T) {
    quotient_unary_respects(qrel, f) and qrel.rel(a, b) implies
    quotient_over_unary_op(qrel, f, quotient_over_mk(qrel, a)) =
        quotient_over_unary_op(qrel, f, quotient_over_mk(qrel, b))
} by {
    if quotient_unary_respects(qrel, f) and qrel.rel(a, b) {
        quotient_over_unary_op_projection(qrel, f, a)
        quotient_over_unary_op_projection(qrel, f, b)
        quotient_unary_respects(qrel, f) = forall(x: T, y: T) {
            qrel.rel(x, y) implies qrel.rel(f(x), f(y))
        }
        qrel.rel(f(a), f(b))
        quotient_over_mk_eq_of_rel(qrel, f(a), f(b))
    }
}

/// The quotient-over value obtained by applying a binary representative operation.
let quotient_over_binary_op_at[T](op: (T, T) -> T, p: QuotientOver[T], q: QuotientOver[T]) -> result: QuotientOver[T] satisfy {
    exists(a: T, b: T) {
        quotient_over_mk(p.qrel, a) = p and quotient_over_mk(q.qrel, b) = q and
        result = quotient_over_mk(p.qrel, op(a, b))
    }
} by {
    quotient_over_mk_surjective(p)
    let a: T satisfy {
        quotient_over_mk(p.qrel, a) = p
    }
    quotient_over_mk_surjective(q)
    let b: T satisfy {
        quotient_over_mk(q.qrel, b) = q
    }
}

/// The quotient-over binary operation has a representative description.
theorem quotient_over_binary_op_at_has_representatives[T](
    op: (T, T) -> T,
    p: QuotientOver[T],
    q: QuotientOver[T]
) {
    exists(a: T, b: T) {
        quotient_over_mk(p.qrel, a) = p and quotient_over_mk(q.qrel, b) = q and
        quotient_over_binary_op_at(op, p, q) = quotient_over_mk(p.qrel, op(a, b))
    }
}

/// The quotient-over binary operation induced by a representative operation.
define quotient_over_binary_op[T](op: (T, T) -> T) -> (QuotientOver[T], QuotientOver[T]) -> QuotientOver[T] {
    function(p: QuotientOver[T], q: QuotientOver[T]) {
        quotient_over_binary_op_at(op, p, q)
    }
}

/// The quotient-over binary operation agrees with the representative operation on projections.
theorem quotient_over_binary_op_projection[T](qrel: QuotientRelation[T], op: (T, T) -> T, a: T, b: T) {
    quotient_binary_respects(qrel, op) implies
    quotient_over_binary_op(op, quotient_over_mk(qrel, a), quotient_over_mk(qrel, b)) =
        quotient_over_mk(qrel, op(a, b))
} by {
    if quotient_binary_respects(qrel, op) {
        quotient_over_binary_op_at_has_representatives(
            op,
            quotient_over_mk(qrel, a),
            quotient_over_mk(qrel, b)
        )
        quotient_over_mk_fields(qrel, a)
        quotient_over_mk_fields(qrel, b)
        let (x: T, y: T) satisfy {
            quotient_over_mk(quotient_over_mk(qrel, a).qrel, x) = quotient_over_mk(qrel, a) and
            quotient_over_mk(quotient_over_mk(qrel, b).qrel, y) = quotient_over_mk(qrel, b) and
            quotient_over_binary_op_at(op, quotient_over_mk(qrel, a), quotient_over_mk(qrel, b)) =
                quotient_over_mk(quotient_over_mk(qrel, a).qrel, op(x, y))
        }
        rel_of_quotient_over_mk_eq(qrel, x, a)
        rel_of_quotient_over_mk_eq(qrel, y, b)
        qrel.rel(x, a)
        qrel.rel(y, b)
        quotient_binary_respects(qrel, op) = forall(x1: T, x2: T, y1: T, y2: T) {
            qrel.rel(x1, x2) and qrel.rel(y1, y2) implies
            qrel.rel(op(x1, y1), op(x2, y2))
        }
        quotient_over_mk_eq_of_rel(qrel, op(x, y), op(a, b))
        quotient_over_mk(qrel, op(x, y)) = quotient_over_mk(qrel, op(a, b))
        quotient_over_binary_op(op, quotient_over_mk(qrel, a), quotient_over_mk(qrel, b)) =
            quotient_over_mk(qrel, op(a, b))
    }
}

/// A relation-preserving binary operation on quotient-over elements respects changes on the left.
theorem quotient_over_binary_op_projection_left[T](
    qrel: QuotientRelation[T],
    op: (T, T) -> T,
    a1: T,
    a2: T,
    b: T
) {
    quotient_binary_respects(qrel, op) and qrel.rel(a1, a2) implies
    quotient_over_binary_op(op, quotient_over_mk(qrel, a1), quotient_over_mk(qrel, b)) =
        quotient_over_binary_op(op, quotient_over_mk(qrel, a2), quotient_over_mk(qrel, b))
} by {
    if quotient_binary_respects(qrel, op) and qrel.rel(a1, a2) {
        quotient_over_binary_op_projection(qrel, op, a1, b)
        quotient_over_binary_op_projection(qrel, op, a2, b)
        quotient_binary_respects(qrel, op) = forall(x1: T, x2: T, y1: T, y2: T) {
            qrel.rel(x1, x2) and qrel.rel(y1, y2) implies
            qrel.rel(op(x1, y1), op(x2, y2))
        }
        qrel.rel(op(a1, b), op(a2, b))
        quotient_over_mk_eq_of_rel(qrel, op(a1, b), op(a2, b))
    }
}

/// A relation-preserving binary operation on quotient-over elements respects changes on the right.
theorem quotient_over_binary_op_projection_right[T](
    qrel: QuotientRelation[T],
    op: (T, T) -> T,
    a: T,
    b1: T,
    b2: T
) {
    quotient_binary_respects(qrel, op) and qrel.rel(b1, b2) implies
    quotient_over_binary_op(op, quotient_over_mk(qrel, a), quotient_over_mk(qrel, b1)) =
        quotient_over_binary_op(op, quotient_over_mk(qrel, a), quotient_over_mk(qrel, b2))
} by {
    if quotient_binary_respects(qrel, op) and qrel.rel(b1, b2) {
        quotient_over_binary_op_projection(qrel, op, a, b1)
        quotient_over_binary_op_projection(qrel, op, a, b2)
        quotient_binary_respects(qrel, op) = forall(x1: T, x2: T, y1: T, y2: T) {
            qrel.rel(x1, x2) and qrel.rel(y1, y2) implies
            qrel.rel(op(x1, y1), op(x2, y2))
        }
        qrel.rel(a, a)
        qrel.rel(op(a, b1), op(a, b2))
        quotient_over_mk_eq_of_rel(qrel, op(a, b1), op(a, b2))
    }
}

/// A left identity representative gives a left identity on projected quotient-over elements.
theorem quotient_over_binary_op_projection_identity_left[T](
    qrel: QuotientRelation[T],
    op: (T, T) -> T,
    e: T,
    a: T
) {
    quotient_binary_respects(qrel, op) and op(e, a) = a implies
    quotient_over_binary_op(op, quotient_over_const(qrel, e), quotient_over_mk(qrel, a)) =
        quotient_over_mk(qrel, a)
} by {
    if quotient_binary_respects(qrel, op) and op(e, a) = a {
        quotient_over_const_projection(qrel, e)
        quotient_over_binary_op_projection(qrel, op, e, a)
    }
}

/// A right identity representative gives a right identity on projected quotient-over elements.
theorem quotient_over_binary_op_projection_identity_right[T](
    qrel: QuotientRelation[T],
    op: (T, T) -> T,
    e: T,
    a: T
) {
    quotient_binary_respects(qrel, op) and op(a, e) = a implies
    quotient_over_binary_op(op, quotient_over_mk(qrel, a), quotient_over_const(qrel, e)) =
        quotient_over_mk(qrel, a)
} by {
    if quotient_binary_respects(qrel, op) and op(a, e) = a {
        quotient_over_const_projection(qrel, e)
        quotient_over_binary_op_projection(qrel, op, a, e)
    }
}

/// A quotient-respecting function is well-defined on equal projections.
theorem quotient_respects_well_defined[T, U](qrel: QuotientRelation[T], f: T -> U, a: T, b: T) {
    quotient_respects(qrel, f) and
    quotient_projection(qrel, a) = quotient_projection(qrel, b) implies f(a) = f(b)
} by {
    if quotient_respects(qrel, f) and
        quotient_projection(qrel, a) = quotient_projection(qrel, b) {
        rel_of_quotient_projection_eq(qrel, a, b)
        quotient_respects(qrel, f) = forall(x: T, y: T) {
            qrel.rel(x, y) implies f(x) = f(y)
        }
        f(a) = f(b)
    }
}

/// The value obtained by lifting a representative function into a plain type at a quotient-over element.
let quotient_over_lift_to_at[T, U](f: T -> U, q: QuotientOver[T]) -> result: U satisfy {
    exists(a: T) {
        quotient_over_mk(q.qrel, a) = q and result = f(a)
    }
} by {
    quotient_over_mk_surjective(q)
    let a: T satisfy {
        quotient_over_mk(q.qrel, a) = q
    }
}

/// The lifted-into-a-plain-type value has a representative description.
theorem quotient_over_lift_to_at_has_representative[T, U](
    f: T -> U,
    q: QuotientOver[T]
) {
    exists(a: T) {
        quotient_over_mk(q.qrel, a) = q and quotient_over_lift_to_at(f, q) = f(a)
    }
}

/// The lifted-into-a-plain-type value agrees with the representative function on projections.
theorem quotient_over_lift_to_at_projection[T, U](
    qrel: QuotientRelation[T],
    f: T -> U,
    a: T
) {
    quotient_respects(qrel, f) implies
    quotient_over_lift_to_at(f, quotient_over_mk(qrel, a)) = f(a)
} by {
    if quotient_respects(qrel, f) {
        quotient_over_lift_to_at_has_representative(f, quotient_over_mk(qrel, a))
        let b: T satisfy {
            quotient_over_mk(qrel, b) = quotient_over_mk(qrel, a) and
            quotient_over_lift_to_at(f, quotient_over_mk(qrel, a)) = f(b)
        }
        rel_of_quotient_over_mk_eq(qrel, b, a)
        qrel.rel(b, a)
        quotient_respects_well_defined(qrel, f, b, a)
        f(b) = f(a)
        quotient_over_lift_to_at(f, quotient_over_mk(qrel, a)) = f(a)
    }
}

/// The function from quotient-over elements into a plain type induced by a representative function.
define quotient_over_lift_to[T, U](f: T -> U) -> QuotientOver[T] -> U {
    function(q: QuotientOver[T]) {
        quotient_over_lift_to_at(f, q)
    }
}

/// The quotient-over lift into a plain type agrees with the representative function on projections.
theorem quotient_over_lift_to_projection[T, U](
    qrel: QuotientRelation[T],
    f: T -> U,
    a: T
) {
    quotient_respects(qrel, f) implies
    quotient_over_lift_to(f, quotient_over_mk(qrel, a)) = f(a)
} by {
    if quotient_respects(qrel, f) {
        quotient_over_lift_to(f, quotient_over_mk(qrel, a)) =
            quotient_over_lift_to_at(f, quotient_over_mk(qrel, a))
        quotient_over_lift_to_at_projection(qrel, f, a)
    }
}

/// The value obtained by lifting a representative function to a quotient element.
let quotient_lift_at[T, U: Inhabited](qrel: QuotientRelation[T], f: T -> U, q: Quotient[T]) -> result: U satisfy {
    q.rel = qrel.rel implies exists(a: T) {
        quotient_projection(qrel, a) = q and result = f(a)
    }
} by {
    if q.rel = qrel.rel {
        quotient_projection_surjective_on_relation(qrel, q)
        let a: T satisfy {
            quotient_projection(qrel, a) = q
        }
        exists(a0: T) {
            quotient_projection(qrel, a0) = q and f(a) = f(a0)
        }
    }
}

/// The lifted value has a representative description.
theorem quotient_lift_at_has_representative[T, U: Inhabited](
    qrel: QuotientRelation[T],
    f: T -> U,
    q: Quotient[T]
) {
    q.rel = qrel.rel implies exists(a: T) {
        quotient_projection(qrel, a) = q and quotient_lift_at(qrel, f, q) = f(a)
    }
}

/// The lifted value agrees with a representative function on quotient projections.
theorem quotient_lift_at_projection[T, U: Inhabited](qrel: QuotientRelation[T], f: T -> U, a: T) {
    quotient_respects(qrel, f) implies
    quotient_lift_at(qrel, f, quotient_projection(qrel, a)) = f(a)
} by {
    if quotient_respects(qrel, f) {
        quotient_mk_fields(qrel, a)
        quotient_lift_at_has_representative(qrel, f, quotient_projection(qrel, a))
        let b: T satisfy {
            quotient_projection(qrel, b) = quotient_projection(qrel, a) and
            quotient_lift_at(qrel, f, quotient_projection(qrel, a)) = f(b)
        }
        quotient_respects_well_defined(qrel, f, b, a)
        quotient_lift_at(qrel, f, quotient_projection(qrel, a)) = f(a)
    }
}

/// The function on a quotient induced by a relation-invariant representative function.
define quotient_lift[T, U: Inhabited](qrel: QuotientRelation[T], f: T -> U) -> Quotient[T] -> U {
    function(q: Quotient[T]) {
        quotient_lift_at(qrel, f, q)
    }
}

/// The quotient lift agrees with the representative function on projections.
theorem quotient_lift_projection[T, U: Inhabited](qrel: QuotientRelation[T], f: T -> U, a: T) {
    quotient_respects(qrel, f) implies quotient_lift(qrel, f, quotient_projection(qrel, a)) = f(a)
} by {
    if quotient_respects(qrel, f) {
        quotient_lift(qrel, f, quotient_projection(qrel, a)) =
            quotient_lift_at(qrel, f, quotient_projection(qrel, a))
        quotient_lift_at_projection(qrel, f, a)
    }
}

/// A quotient function is determined on one element by its values on representatives.
theorem quotient_function_eq_at_projection[T, U](
    qrel: QuotientRelation[T],
    f: Quotient[T] -> U,
    g: Quotient[T] -> U,
    q: Quotient[T]
) {
    q.rel = qrel.rel and
    (forall(a: T) { f(quotient_projection(qrel, a)) = g(quotient_projection(qrel, a)) })
    implies f(q) = g(q)
} by {
    if q.rel = qrel.rel and
        (forall(a: T) { f(quotient_projection(qrel, a)) = g(quotient_projection(qrel, a)) }) {
        quotient_projection_surjective_on_relation(qrel, q)
        let a: T satisfy {
            quotient_projection(qrel, a) = q
        }
        f(quotient_projection(qrel, a)) = g(quotient_projection(qrel, a))
        f(q) = g(q)
    }
}

/// Two quotient functions agree on all quotient elements over a relation when they agree on projections.
theorem quotient_function_eq_on_relation[T, U](
    qrel: QuotientRelation[T],
    f: Quotient[T] -> U,
    g: Quotient[T] -> U
) {
    (forall(a: T) { f(quotient_projection(qrel, a)) = g(quotient_projection(qrel, a)) })
    implies forall(q: Quotient[T]) {
        q.rel = qrel.rel implies f(q) = g(q)
    }
} by {
    if forall(a: T) { f(quotient_projection(qrel, a)) = g(quotient_projection(qrel, a)) } {
        forall(q: Quotient[T]) {
            if q.rel = qrel.rel {
                quotient_function_eq_at_projection(qrel, f, g, q)
                f(q) = g(q)
            }
        }
    }
}

/// A quotient lift has the unique prescribed value on each projection.
theorem quotient_lift_unique_at_projection[T, U: Inhabited](
    qrel: QuotientRelation[T],
    f: T -> U,
    g: Quotient[T] -> U,
    a: T
) {
    quotient_respects(qrel, f) and
    (forall(x: T) { g(quotient_projection(qrel, x)) = f(x) }) implies
    g(quotient_projection(qrel, a)) = quotient_lift(qrel, f, quotient_projection(qrel, a))
} by {
    if quotient_respects(qrel, f) and
        forall(x: T) { g(quotient_projection(qrel, x)) = f(x) } {
        quotient_lift_projection(qrel, f, a)
        g(quotient_projection(qrel, a)) = quotient_lift(qrel, f, quotient_projection(qrel, a))
    }
}

/// A quotient lift is the unique quotient function with the prescribed representative values on the relation.
theorem quotient_lift_unique_on_relation[T, U: Inhabited](
    qrel: QuotientRelation[T],
    f: T -> U,
    g: Quotient[T] -> U
) {
    quotient_respects(qrel, f) and
    (forall(x: T) { g(quotient_projection(qrel, x)) = f(x) }) implies
    forall(q: Quotient[T]) {
        q.rel = qrel.rel implies g(q) = quotient_lift(qrel, f, q)
    }
} by {
    if quotient_respects(qrel, f) and
        forall(x: T) { g(quotient_projection(qrel, x)) = f(x) } {
        forall(a: T) {
            quotient_lift_unique_at_projection(qrel, f, g, a)
            g(quotient_projection(qrel, a)) = quotient_lift(qrel, f, quotient_projection(qrel, a))
        }
        forall(q: Quotient[T]) {
            if q.rel = qrel.rel {
                quotient_function_eq_at_projection(qrel, g, quotient_lift(qrel, f), q)
                g(q) = quotient_lift(qrel, f, q)
            }
        }
    }
}

/// The membership predicate for the quotient image of a representative subset.
define quotient_image_set_pred[T](qrel: QuotientRelation[T], s: Set[T], q: Quotient[T]) -> Bool {
    exists(a: T) {
        s.contains(a) and quotient_projection(qrel, a) = q
    }
}

/// The image of a representative subset under the quotient projection.
define quotient_image_set[T](qrel: QuotientRelation[T], s: Set[T]) -> Set[Quotient[T]] {
    Set[Quotient[T]].new(quotient_image_set_pred(qrel, s))
}

/// The membership predicate for the quotient preimage of a subset of quotients.
define quotient_preimage_set_pred[T](qrel: QuotientRelation[T], qs: Set[Quotient[T]], a: T) -> Bool {
    qs.contains(quotient_projection(qrel, a))
}

/// The preimage of a quotient subset under the quotient projection.
define quotient_preimage_set[T](qrel: QuotientRelation[T], qs: Set[Quotient[T]]) -> Set[T] {
    Set[T].new(quotient_preimage_set_pred(qrel, qs))
}

/// Membership in the quotient image is witnessed by a representative.
theorem quotient_image_set_contains_eq[T](qrel: QuotientRelation[T], s: Set[T], q: Quotient[T]) {
    quotient_image_set(qrel, s).contains(q) = exists(a: T) {
        s.contains(a) and quotient_projection(qrel, a) = q
    }
}

/// A representative subset member projects into the quotient image.
theorem quotient_image_set_contains_projection[T](qrel: QuotientRelation[T], s: Set[T], a: T) {
    s.contains(a) implies quotient_image_set(qrel, s).contains(quotient_projection(qrel, a))
} by {
    if s.contains(a) {
        quotient_image_set_contains_eq(qrel, s, quotient_projection(qrel, a))
        quotient_image_set(qrel, s).contains(quotient_projection(qrel, a))
    }
}

/// Membership in the quotient preimage is given by membership of the projection.
theorem quotient_preimage_set_contains_eq[T](qrel: QuotientRelation[T], qs: Set[Quotient[T]], a: T) {
    quotient_preimage_set(qrel, qs).contains(a) = qs.contains(quotient_projection(qrel, a))
}

/// The preimage of a quotient subset is saturated under the underlying relation.
theorem quotient_preimage_set_is_saturated[T](qrel: QuotientRelation[T], qs: Set[Quotient[T]]) {
    is_saturated_under(quotient_preimage_set(qrel, qs), qrel.rel)
} by {
    forall(x: T, y: T) {
        if quotient_preimage_set(qrel, qs).contains(x) and qrel.rel(x, y) {
            quotient_preimage_set_contains_eq(qrel, qs, x)
            quotient_projection_eq_of_rel(qrel, x, y)
            quotient_preimage_set_contains_eq(qrel, qs, y)
            quotient_preimage_set(qrel, qs).contains(y)
        }
    }
}

/// Membership of `x` in a saturated set propagates along the relation.
theorem saturated_step[T](s: Set[T], r: (T, T) -> Bool, x: T, y: T) {
    is_saturated_under(s, r) and s.contains(x) and r(x, y) implies s.contains(y)
} by {
    if is_saturated_under(s, r) and s.contains(x) and r(x, y) {
        is_saturated_under(s, r) = forall(y1: T, y2: T) {
            s.contains(y1) and r(y1, y2) implies s.contains(y2)
        }
        s.contains(y)
    }
}

/// A saturated representative subset contains every representative whose projection is in its quotient image.
theorem saturated_contains_of_quotient_image_contains_projection[T](qrel: QuotientRelation[T], s: Set[T], a: T) {
    is_saturated_under(s, qrel.rel) and quotient_image_set(qrel, s).contains(quotient_projection(qrel, a))
    implies s.contains(a)
} by {
    if is_saturated_under(s, qrel.rel) and quotient_image_set(qrel, s).contains(quotient_projection(qrel, a)) {
        quotient_image_set_contains_eq(qrel, s, quotient_projection(qrel, a))
        let b: T satisfy {
            s.contains(b) and quotient_projection(qrel, b) = quotient_projection(qrel, a)
        }
        rel_of_quotient_projection_eq(qrel, b, a)
        saturated_step(s, qrel.rel, b, a)
        s.contains(a)
    }
}

/// For saturated representative subsets, quotient-image membership of a projection is original membership.
theorem saturated_quotient_image_projection_contains_iff[T](qrel: QuotientRelation[T], s: Set[T], a: T) {
    is_saturated_under(s, qrel.rel) implies
        quotient_image_set(qrel, s).contains(quotient_projection(qrel, a)) = s.contains(a)
} by {
    if is_saturated_under(s, qrel.rel) {
        if s.contains(a) {
            quotient_image_set_contains_projection(qrel, s, a)
            quotient_image_set(qrel, s).contains(quotient_projection(qrel, a))
        }
        if quotient_image_set(qrel, s).contains(quotient_projection(qrel, a)) {
            saturated_contains_of_quotient_image_contains_projection(qrel, s, a)
            s.contains(a)
        }
    }
}

/// Membership in the preimage of the image is witnessed by a representative.
theorem quotient_preimage_image_contains_eq[T](qrel: QuotientRelation[T], s: Set[T], x: T) {
    quotient_preimage_set(qrel, quotient_image_set(qrel, s)).contains(x) = exists(a: T) {
        s.contains(a) and quotient_projection(qrel, a) = quotient_projection(qrel, x)
    }
} by {
    quotient_preimage_set_contains_eq(qrel, quotient_image_set(qrel, s), x)
    quotient_image_set_contains_eq(qrel, s, quotient_projection(qrel, x))
}

/// The image of the preimage of any quotient subset is contained in the original.
theorem quotient_image_preimage_subset[T](qrel: QuotientRelation[T], qs: Set[Quotient[T]]) {
    forall(q: Quotient[T]) {
        quotient_image_set(qrel, quotient_preimage_set(qrel, qs)).contains(q) implies qs.contains(q)
    }
} by {
    forall(q: Quotient[T]) {
        if quotient_image_set(qrel, quotient_preimage_set(qrel, qs)).contains(q) {
            quotient_image_set_contains_eq(qrel, quotient_preimage_set(qrel, qs), q)
            let a: T satisfy {
                quotient_preimage_set(qrel, qs).contains(a) and quotient_projection(qrel, a) = q
            }
            quotient_preimage_set_contains_eq(qrel, qs, a)
            qs.contains(q)
        }
    }
}

/// The original set is contained in the preimage of its image.
theorem quotient_subset_preimage_image[T](qrel: QuotientRelation[T], s: Set[T]) {
    forall(x: T) {
        s.contains(x) implies quotient_preimage_set(qrel, quotient_image_set(qrel, s)).contains(x)
    }
} by {
    forall(x: T) {
        if s.contains(x) {
            quotient_image_set_contains_eq(qrel, s, quotient_projection(qrel, x))
            quotient_preimage_set_contains_eq(qrel, quotient_image_set(qrel, s), x)
            quotient_preimage_set(qrel, quotient_image_set(qrel, s)).contains(x)
        }
    }
}

/// For saturated representative subsets, the preimage of the image is contained in the original.
theorem saturated_preimage_image_subset[T](qrel: QuotientRelation[T], s: Set[T]) {
    is_saturated_under(s, qrel.rel) implies
    forall(x: T) {
        quotient_preimage_set(qrel, quotient_image_set(qrel, s)).contains(x) implies s.contains(x)
    }
} by {
    if is_saturated_under(s, qrel.rel) {
        forall(x: T) {
            if quotient_preimage_set(qrel, quotient_image_set(qrel, s)).contains(x) {
                quotient_preimage_image_contains_eq(qrel, s, x)
                let a: T satisfy {
                    s.contains(a) and quotient_projection(qrel, a) = quotient_projection(qrel, x)
                }
                rel_of_quotient_projection_eq(qrel, a, x)
                saturated_step(s, qrel.rel, a, x)
                s.contains(x)
            }
        }
    }
}

/// Pointwise membership equality for the preimage-image of a saturated set.
theorem saturated_preimage_image_iff[T](qrel: QuotientRelation[T], s: Set[T], x: T) {
    is_saturated_under(s, qrel.rel) implies
        quotient_preimage_set(qrel, quotient_image_set(qrel, s)).contains(x) = s.contains(x)
} by {
    if is_saturated_under(s, qrel.rel) {
        if s.contains(x) {
            quotient_subset_preimage_image(qrel, s)
            quotient_preimage_set(qrel, quotient_image_set(qrel, s)).contains(x)
        }
        if quotient_preimage_set(qrel, quotient_image_set(qrel, s)).contains(x) {
            saturated_preimage_image_subset(qrel, s)
            s.contains(x)
        }
    }
}

/// The preimage of the image of a saturated set recovers the original set.
theorem quotient_preimage_image_of_saturated[T](qrel: QuotientRelation[T], s: Set[T]) {
    is_saturated_under(s, qrel.rel) implies
        quotient_preimage_set(qrel, quotient_image_set(qrel, s)) = s
} by {
    if is_saturated_under(s, qrel.rel) {
        let u = quotient_preimage_set(qrel, quotient_image_set(qrel, s))
        forall(x: T) {
            saturated_preimage_image_iff(qrel, s, x)
            u.contains(x) = s.contains(x)
        }
        set_ext(u, s)
    }
}

/// A subset of `qrel`-classes is contained in the image of its preimage.
theorem on_relation_subset_image_preimage[T](qrel: QuotientRelation[T], qs: Set[Quotient[T]]) {
    forall(q: Quotient[T]) {
        qs.contains(q) and q.rel = qrel.rel implies
            quotient_image_set(qrel, quotient_preimage_set(qrel, qs)).contains(q)
    }
} by {
    forall(q: Quotient[T]) {
        if qs.contains(q) and q.rel = qrel.rel {
            quotient_projection_surjective_on_relation(qrel, q)
            let a: T satisfy {
                quotient_projection(qrel, a) = q
            }
            quotient_preimage_set_contains_eq(qrel, qs, a)
            quotient_image_set_contains_eq(qrel, quotient_preimage_set(qrel, qs), q)
            quotient_image_set(qrel, quotient_preimage_set(qrel, qs)).contains(q)
        }
    }
}

/// Combined pointwise equivalence: membership of qrel-classes in qs equals membership in the image of the preimage.
theorem on_relation_image_preimage_iff[T](qrel: QuotientRelation[T], qs: Set[Quotient[T]], q: Quotient[T]) {
    (qs.contains(q) implies q.rel = qrel.rel)
    implies quotient_image_set(qrel, quotient_preimage_set(qrel, qs)).contains(q) = qs.contains(q)
} by {
    if qs.contains(q) implies q.rel = qrel.rel {
        if qs.contains(q) {
            quotient_projection_surjective_on_relation(qrel, q)
            let a: T satisfy {
                quotient_projection(qrel, a) = q
            }
            quotient_preimage_set_contains_eq(qrel, qs, a)
            quotient_image_set_contains_eq(qrel, quotient_preimage_set(qrel, qs), q)
            quotient_image_set(qrel, quotient_preimage_set(qrel, qs)).contains(q)
        }
        if quotient_image_set(qrel, quotient_preimage_set(qrel, qs)).contains(q) {
            quotient_image_set_contains_eq(qrel, quotient_preimage_set(qrel, qs), q)
            let a: T satisfy {
                quotient_preimage_set(qrel, qs).contains(a) and quotient_projection(qrel, a) = q
            }
            quotient_preimage_set_contains_eq(qrel, qs, a)
            qs.contains(q)
        }
    }
}

/// The image of the preimage of a quotient subset of `qrel`-classes recovers it.
theorem quotient_image_preimage_on_relation[T](qrel: QuotientRelation[T], qs: Set[Quotient[T]]) {
    (forall(q: Quotient[T]) { qs.contains(q) implies q.rel = qrel.rel })
    implies quotient_image_set(qrel, quotient_preimage_set(qrel, qs)) = qs
} by {
    if forall(q: Quotient[T]) { qs.contains(q) implies q.rel = qrel.rel } {
        let u = quotient_image_set(qrel, quotient_preimage_set(qrel, qs))
        forall(q: Quotient[T]) {
            on_relation_image_preimage_iff(qrel, qs, q)
            u.contains(q) = qs.contains(q)
        }
        set_ext(u, qs)
    }
}

/// The kernel of a function: representatives are related when they share an image.
define kernel_relation[T, U](f: T -> U, x: T, y: T) -> Bool {
    f(x) = f(y)
}

/// The kernel relation is reflexive.
theorem kernel_relation_is_reflexive[T, U](f: T -> U) {
    is_reflexive(kernel_relation(f))
} by {
    forall(x: T) {
        f(x) = f(x)
        kernel_relation(f, x, x)
    }
}

/// The kernel relation is symmetric.
theorem kernel_relation_is_symmetric[T, U](f: T -> U) {
    is_symmetric(kernel_relation(f))
} by {
    forall(x: T, y: T) {
        if kernel_relation(f, x, y) {
            kernel_relation(f, y, x)
        }
    }
}

/// The kernel relation is transitive.
theorem kernel_relation_is_transitive[T, U](f: T -> U) {
    is_transitive(kernel_relation(f))
} by {
    forall(x: T, y: T, z: T) {
        if kernel_relation(f, x, y) and kernel_relation(f, y, z) {
            f(y) = f(z)
            f(x) = f(z)
            kernel_relation(f, x, z)
        }
    }
}

/// The kernel relation is an equivalence relation.
theorem kernel_relation_is_equivalence[T, U](f: T -> U) {
    is_equivalence(kernel_relation(f))
} by {
    kernel_relation_is_reflexive(f)
    kernel_relation_is_symmetric(f)
    kernel_relation_is_transitive(f)
}

/// The quotient relation associated to the kernel of a function.
let kernel_quotient_relation[T, U](f: T -> U) -> result: QuotientRelation[T] satisfy {
    QuotientRelation[T].new(kernel_relation(f)) = Option.some(result)
} by {
    kernel_relation_is_equivalence(f)
}

/// The kernel quotient relation has the kernel relation as its underlying relation.
theorem kernel_quotient_relation_rel[T, U](f: T -> U) {
    kernel_quotient_relation(f).rel = kernel_relation(f)
} by {
    quotient_relation_new_round_trip(kernel_relation(f), kernel_quotient_relation(f))
}

/// Every function respects its own kernel relation.
theorem kernel_relation_respects[T, U](f: T -> U) {
    quotient_respects(kernel_quotient_relation(f), f)
} by {
    forall(a: T, b: T) {
        if kernel_quotient_relation(f).rel(a, b) {
            kernel_quotient_relation_rel(f)
            f(a) = f(b)
        }
    }
}

/// The function on the kernel quotient that factors a function through its kernel.
define kernel_lift[T, U: Inhabited](f: T -> U) -> Quotient[T] -> U {
    quotient_lift(kernel_quotient_relation(f), f)
}

/// The kernel lift agrees with the original function on projected representatives.
theorem kernel_lift_projection[T, U: Inhabited](f: T -> U, a: T) {
    kernel_lift(f, quotient_projection(kernel_quotient_relation(f), a)) = f(a)
} by {
    kernel_relation_respects(f)
    quotient_lift_projection(kernel_quotient_relation(f), f, a)
}

/// Equality of kernel projections is equality of the function values.
theorem kernel_quotient_projection_eq_iff[T, U](f: T -> U, a: T, b: T) {
    (quotient_projection(kernel_quotient_relation(f), a) = quotient_projection(kernel_quotient_relation(f), b))
    = (f(a) = f(b))
} by {
    quotient_projection_eq_iff_rel(kernel_quotient_relation(f), a, b)
    kernel_quotient_relation_rel(f)
}

/// On kernel quotients, equality of kernel-lift values forces equality of quotient elements.
theorem kernel_lift_eq_imp_quotient_eq[T, U: Inhabited](f: T -> U, q: Quotient[T], r: Quotient[T]) {
    q.rel = kernel_quotient_relation(f).rel and
    r.rel = kernel_quotient_relation(f).rel and
    kernel_lift(f, q) = kernel_lift(f, r) implies q = r
} by {
    if q.rel = kernel_quotient_relation(f).rel and
        r.rel = kernel_quotient_relation(f).rel and
        kernel_lift(f, q) = kernel_lift(f, r) {
        quotient_projection_surjective_on_relation(kernel_quotient_relation(f), q)
        let a: T satisfy {
            quotient_projection(kernel_quotient_relation(f), a) = q
        }
        quotient_projection_surjective_on_relation(kernel_quotient_relation(f), r)
        let b: T satisfy {
            quotient_projection(kernel_quotient_relation(f), b) = r
        }
        kernel_lift_projection(f, a)
        kernel_lift_projection(f, b)
        kernel_quotient_projection_eq_iff(f, a, b)
        quotient_projection(kernel_quotient_relation(f), a) = quotient_projection(kernel_quotient_relation(f), b)
        q = r
    }
}

/// A surjective representative function has a kernel-quotient preimage for every target value.
theorem kernel_lift_surjective_of_surjective[T, U: Inhabited](f: T -> U) {
    is_surjective_fn(f) implies forall(y: U) {
        exists(q: Quotient[T]) {
            q.rel = kernel_quotient_relation(f).rel and kernel_lift(f, q) = y
        }
    }
} by {
    if is_surjective_fn(f) {
        forall(y: U) {
            surjective_fn_has_preimage(f, y)
            let a: T satisfy {
                f(a) = y
            }
            quotient_projection_eq_mk(kernel_quotient_relation(f), a)
            quotient_mk_fields(kernel_quotient_relation(f), a)
            kernel_lift_projection(f, a)
            exists(q: Quotient[T]) {
                q.rel = kernel_quotient_relation(f).rel and kernel_lift(f, q) = y
            }
        }
    }
}
