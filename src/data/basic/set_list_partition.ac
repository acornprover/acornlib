from list import List, map, sum, map_contains, map_contains_of_contains,
    unique_implies_tail_unique, unique_implies_no_duplicate, list_contains_implies_count_geq_one
from nat import Nat
from nat import sum_lte
from data.basic.set import Set, cardinality_is_smallest_cardinality, disjoint_union_is_length,
    empty_set_contains_eq, union_contains_cases, union_is_at_most_length, set_ext,
    list_indexed_union, list_indexed_union_contains_of_contains,
    list_indexed_union_contains_witness

/// The union of a list of sets, formed recursively from the head and tail.
define set_list_union[K](items: List[Set[K]]) -> Set[K] {
    match items {
        List.nil {
            Set[K].empty_set
        }
        List.cons(head, tail) {
            head.union(set_list_union[K](tail))
        }
    }
}

lemma set_list_union_contains_nil_member[K](c: Set[K], x: K) { List.nil[Set[K]].contains(c) and c.contains(x) implies set_list_union[K](List.nil[Set[K]]).contains(x) } by {
    false
}

/// The head of a cons list is contained in the recursive list union.
theorem set_list_union_contains_head[K](head: Set[K], tail: List[Set[K]], x: K) { head.contains(x) implies set_list_union[K](List.cons[Set[K]](head, tail)).contains(x) } by {
    set_list_union[K](List.cons[Set[K]](head, tail)) = head.union(set_list_union[K](tail))
    head.union(set_list_union[K](tail)).contains(x)
}

/// The recursive union of the tail is contained in the recursive union of the cons list.
theorem set_list_union_contains_tail[K](head: Set[K], tail: List[Set[K]], x: K) { set_list_union[K](tail).contains(x) implies set_list_union[K](List.cons[Set[K]](head, tail)).contains(x) } by {
    set_list_union[K](List.cons[Set[K]](head, tail)) = head.union(set_list_union[K](tail))
    head.union(set_list_union[K](tail)).contains(x)
}

lemma cons_contains_tail_of_contains_not_head[A](head: A, tail: List[A], item: A) { List.cons[A](head, tail).contains(item) and head != item implies tail.contains(item) } by {
    List.cons[A](head, tail).contains(item) = (if head = item { true } else { tail.contains(item) })
    if head = item {
        false
    }
    tail.contains(item)
}

lemma set_list_union_contains_cons_step_head_eq[K](head: Set[K], tail: List[Set[K]], c: Set[K], x: K) { head = c and c.contains(x) implies set_list_union[K](List.cons[Set[K]](head, tail)).contains(x) } by {
    head.contains(x)
    set_list_union_contains_head[K](head, tail, x)
}

lemma set_list_union_contains_cons_step_head_ne[K](head: Set[K], tail: List[Set[K]], c: Set[K], x: K) { head != c implies ((tail.contains(c) and c.contains(x) implies set_list_union[K](tail).contains(x)) implies (List.cons[Set[K]](head, tail).contains(c) and c.contains(x) implies set_list_union[K](List.cons[Set[K]](head, tail)).contains(x))) } by {
    if tail.contains(c) and c.contains(x) implies set_list_union[K](tail).contains(x) {
        if List.cons[Set[K]](head, tail).contains(c) and c.contains(x) {
            tail.contains(c)
            set_list_union[K](tail).contains(x)
            set_list_union[K](List.cons[Set[K]](head, tail)).contains(x)
        }
    }
}

lemma set_list_union_contains_cons_step[K](head: Set[K], tail: List[Set[K]], c: Set[K], x: K) { (tail.contains(c) and c.contains(x) implies set_list_union[K](tail).contains(x)) implies (List.cons[Set[K]](head, tail).contains(c) and c.contains(x) implies set_list_union[K](List.cons[Set[K]](head, tail)).contains(x)) } by {
    if tail.contains(c) and c.contains(x) implies set_list_union[K](tail).contains(x) {
        if List.cons[Set[K]](head, tail).contains(c) and c.contains(x) {
            if head = c {
                set_list_union_contains_cons_step_head_eq[K](head, tail, c, x)
            } else {
                set_list_union_contains_cons_step_head_ne[K](head, tail, c, x)
            }
        }
    }
}

lemma set_list_union_contains_cons_induction_step[K](head: Set[K], tail: List[Set[K]]) { (forall(d: Set[K], y: K) { tail.contains(d) and d.contains(y) implies set_list_union[K](tail).contains(y) }) implies (forall(d: Set[K], y: K) { List.cons[Set[K]](head, tail).contains(d) and d.contains(y) implies set_list_union[K](List.cons[Set[K]](head, tail)).contains(y) }) } by {
    forall(d: Set[K], y: K) {
        if List.cons[Set[K]](head, tail).contains(d) and d.contains(y) {
            tail.contains(d) and d.contains(y) implies set_list_union[K](tail).contains(y)
            set_list_union_contains_cons_step[K](head, tail, d, y)
            set_list_union[K](List.cons[Set[K]](head, tail)).contains(y)
        }
    }
}

/// A member set contributes all of its elements to the recursive list union.
theorem set_list_union_contains_of_member[K](items: List[Set[K]], c: Set[K], x: K) {
    items.contains(c) and c.contains(x) implies set_list_union[K](items).contains(x)
} by {
    let p: List[Set[K]] -> Bool = function(xs: List[Set[K]]) {
        forall(d: Set[K], y: K) {
            xs.contains(d) and d.contains(y) implies set_list_union[K](xs).contains(y)
        }
    }
    forall(d: Set[K], y: K) {
        set_list_union_contains_nil_member[K](d, y)
    }
    p(List.nil[Set[K]])
    forall(head: Set[K], tail: List[Set[K]]) {
        if p(tail) {
            set_list_union_contains_cons_induction_step[K](head, tail)
            p(List.cons[Set[K]](head, tail))
        }
    }
    List.induction(p)
    forall(xs: List[Set[K]]) { p(xs) }
    p(items)
}

/// Every set in the list has cardinality `n`.
define set_list_all_cardinality_is[K](items: List[Set[K]], n: Nat) -> Bool {
    forall(s: Set[K]) {
        items.contains(s) implies s.cardinality_is(n)
    }
}

/// Every set in the list has the cardinality prescribed by `card`.
define set_list_all_cardinality_by[K](items: List[Set[K]], card: Set[K] -> Nat) -> Bool {
    forall(s: Set[K]) {
        items.contains(s) implies s.cardinality_is(card(s))
    }
}

/// The sum of prescribed cardinalities over a list of sets.
define set_list_cardinality_sum[K](items: List[Set[K]], card: Set[K] -> Nat) -> Nat {
    sum[Nat](map[Set[K], Nat](items, card))
}


/// A recursively disjoint set list has each head disjoint from the union of its tail.
define set_list_recursively_disjoint[K](items: List[Set[K]]) -> Bool {
    match items {
        List.nil {
            true
        }
        List.cons(head, tail) {
            head.is_disjoint(set_list_union[K](tail)) and set_list_recursively_disjoint[K](tail)
        }
    }
}


lemma cons_contains_head[A](head: A, tail: List[A]) { List.cons(head, tail).contains(head) }

/// The tail of a recursively disjoint cons list is recursively disjoint.
theorem set_list_recursively_disjoint_tail[K](head: Set[K], tail: List[Set[K]]) {
    set_list_recursively_disjoint[K](List.cons(head, tail)) implies set_list_recursively_disjoint[K](tail)
} by {
}


/// The head of a recursively disjoint cons list is disjoint from the union of its tail.
theorem set_list_recursively_disjoint_head[K](head: Set[K], tail: List[Set[K]]) {
    set_list_recursively_disjoint[K](List.cons(head, tail)) implies head.is_disjoint(set_list_union[K](tail))
}


lemma mul_cons_length[A](head: A, tail: List[A], n: Nat) { n + n * tail.length = n * List.cons(head, tail).length }


lemma set_list_union_cardinality_recursive_cons_step[K](head: Set[K], tail: List[Set[K]], n: Nat) {
    head.cardinality_is(n) and
    set_list_union[K](tail).cardinality_is(n * tail.length) and
    set_list_recursively_disjoint[K](List.cons(head, tail))
    implies set_list_union[K](List.cons(head, tail)).cardinality_is(n * List.cons(head, tail).length)
} by {
    head.is_disjoint(set_list_union[K](tail))
    head.cardinality_is(n) and set_list_union[K](tail).cardinality_is(n * tail.length) and head.is_disjoint(set_list_union[K](tail))
    disjoint_union_is_length[K](head, set_list_union[K](tail), n, n * tail.length)
    head.union(set_list_union[K](tail)).cardinality_is(n + n * tail.length)
    mul_cons_length[Set[K]](head, tail, n)
    n + n * tail.length = n * List.cons(head, tail).length
    head.union(set_list_union[K](tail)).cardinality_is(n * List.cons(head, tail).length)
    set_list_union[K](List.cons(head, tail)) = head.union(set_list_union[K](tail))
}


lemma set_list_union_cardinality_recursive_nil[K](n: Nat) {
    set_list_all_cardinality_is[K](List.nil[Set[K]], n) and
    set_list_recursively_disjoint[K](List.nil[Set[K]])
    implies set_list_union[K](List.nil[Set[K]]).cardinality_is(n * List.nil[Set[K]].length)
} by {
    Set.empty_set[K].cardinality_is(Nat.0)
    set_list_union[K](List.nil[Set[K]]) = Set.empty_set[K]
    List.nil[Set[K]].length = Nat.0
    n * List.nil[Set[K]].length = Nat.0
}

/// The recursive list union of the empty list has cardinality zero.
theorem set_list_union_nil_cardinality_is_zero[K] {
    set_list_union[K](List.nil[Set[K]]).cardinality_is(Nat.0)
} by {
    set_list_union[K](List.nil[Set[K]]) = Set[K].empty_set
    Set[K].empty_set.cardinality_is(Nat.0)
}


lemma list_forall_contains_elim[A](items: List[A], p: A -> Bool, s: A) { (forall(x: A) { items.contains(x) implies p(x) }) and items.contains(s) implies p(s) }


/// A set in an all-cardinality list has the common cardinality.
theorem set_list_all_cardinality_is_elim[K](items: List[Set[K]], n: Nat, s: Set[K]) {
    set_list_all_cardinality_is[K](items, n) and items.contains(s) implies s.cardinality_is(n)
} by {
    define p(x: Set[K]) -> Bool { x.cardinality_is(n) }
    list_forall_contains_elim[Set[K]](items, p, s)
    p(s)
    s.cardinality_is(n)
}


lemma cons_contains_tail[A](head: A, tail: List[A], item: A) {
    tail.contains(item) implies List.cons(head, tail).contains(item)
}


lemma set_list_all_cardinality_tail_point[K](head: Set[K], tail: List[Set[K]], n: Nat, s: Set[K]) {
    set_list_all_cardinality_is[K](List.cons(head, tail), n) and tail.contains(s) implies s.cardinality_is(n)
} by {
    cons_contains_tail[Set[K]](head, tail, s)
    List.cons(head, tail).contains(s)
    set_list_all_cardinality_is_elim[K](List.cons(head, tail), n, s)
}

lemma set_list_all_cardinality_by_elim[K](items: List[Set[K]], card: Set[K] -> Nat,
    s: Set[K]) {
    set_list_all_cardinality_by[K](items, card) and items.contains(s) implies
    s.cardinality_is(card(s))
} by {
    set_list_all_cardinality_by[K](items, card) = forall(x: Set[K]) {
        items.contains(x) implies x.cardinality_is(card(x))
    }
    forall(x: Set[K]) { items.contains(x) implies x.cardinality_is(card(x)) }
    items.contains(s) implies s.cardinality_is(card(s))
    s.cardinality_is(card(s))
}

lemma set_list_all_cardinality_by_tail_point[K](head: Set[K], tail: List[Set[K]],
    card: Set[K] -> Nat, s: Set[K]) {
    set_list_all_cardinality_by[K](List.cons(head, tail), card) and tail.contains(s)
    implies s.cardinality_is(card(s))
} by {
    cons_contains_tail[Set[K]](head, tail, s)
    List.cons(head, tail).contains(s)
    set_list_all_cardinality_by_elim[K](List.cons(head, tail), card, s)
}

lemma set_list_all_cardinality_by_tail[K](head: Set[K], tail: List[Set[K]],
    card: Set[K] -> Nat) {
    set_list_all_cardinality_by[K](List.cons(head, tail), card) implies
    set_list_all_cardinality_by[K](tail, card)
} by {
    forall(s: Set[K]) {
        if tail.contains(s) {
            set_list_all_cardinality_by_tail_point[K](head, tail, card, s)
            s.cardinality_is(card(s))
        }
    }
}

lemma set_list_all_cardinality_by_head[K](head: Set[K], tail: List[Set[K]],
    card: Set[K] -> Nat) {
    set_list_all_cardinality_by[K](List.cons(head, tail), card) implies
    head.cardinality_is(card(head))
} by {
    cons_contains_head[Set[K]](head, tail)
    set_list_all_cardinality_by_elim[K](List.cons(head, tail), card, head)
}


/// The tail of a cons list inherits the common-cardinality predicate.
theorem set_list_all_cardinality_tail[K](head: Set[K], tail: List[Set[K]], n: Nat) {
    set_list_all_cardinality_is[K](List.cons(head, tail), n) implies
    set_list_all_cardinality_is[K](tail, n)
} by {
    forall(s: Set[K]) {
        if tail.contains(s) {
            set_list_all_cardinality_tail_point[K](head, tail, n, s)
            s.cardinality_is(n)
        }
    }
}


/// The head of an all-cardinality cons list has the common cardinality.
theorem set_list_all_cardinality_head[K](head: Set[K], tail: List[Set[K]], n: Nat) {
    set_list_all_cardinality_is[K](List.cons(head, tail), n) implies head.cardinality_is(n)
} by {
    cons_contains_head[Set[K]](head, tail)
    set_list_all_cardinality_is_elim[K](List.cons(head, tail), n, head)
}


/// A recursively disjoint list of sets with a common finite cardinality has union cardinality `n * length`.
theorem set_list_union_cardinality_recursive[K](items: List[Set[K]], n: Nat) {
    set_list_all_cardinality_is[K](items, n) and
    set_list_recursively_disjoint[K](items)
    implies set_list_union[K](items).cardinality_is(n * items.length)
} by {
    define p(xs: List[Set[K]]) -> Bool {
        set_list_all_cardinality_is[K](xs, n) and
        set_list_recursively_disjoint[K](xs)
        implies set_list_union[K](xs).cardinality_is(n * xs.length)
    }
    set_list_union_cardinality_recursive_nil[K](n)
    p(List.nil[Set[K]])
    forall(head: Set[K], tail: List[Set[K]]) {
        if p(tail) {
            if set_list_all_cardinality_is[K](List.cons(head, tail), n) and set_list_recursively_disjoint[K](List.cons(head, tail)) {
                set_list_all_cardinality_head[K](head, tail, n)
                head.cardinality_is(n)
                set_list_all_cardinality_tail[K](head, tail, n)
                set_list_all_cardinality_is[K](tail, n)
                set_list_recursively_disjoint_tail[K](head, tail)
                set_list_recursively_disjoint[K](tail)
                set_list_union[K](tail).cardinality_is(n * tail.length)
                set_list_union_cardinality_recursive_cons_step[K](head, tail, n)
                set_list_union[K](List.cons(head, tail)).cardinality_is(n * List.cons(head, tail).length)
            }
            p(List.cons(head, tail))
        }
    }
    List.induction(function(xs: List[Set[K]]) { p(xs) })
    forall(xs: List[Set[K]]) { p(xs) }
    p(items)
}

/// A set list is pairwise disjoint when distinct listed sets are disjoint.
define set_list_pairwise_disjoint[K](items: List[Set[K]]) -> Bool {
    forall(a: Set[K], b: Set[K]) {
        items.contains(a) and items.contains(b) and a != b implies a.is_disjoint(b)
    }
}

/// A list of sets covers a set when its recursive union is exactly that set.
define set_list_exactly_covers[K](items: List[Set[K]], whole: Set[K]) -> Bool {
    set_list_union[K](items) = whole
}

/// A finite/list partition is an exact cover by a unique pairwise-disjoint list of sets.
define set_list_partition_of[K](items: List[Set[K]], whole: Set[K]) -> Bool {
    set_list_exactly_covers[K](items, whole) and items.is_unique and set_list_pairwise_disjoint[K](items)
}

lemma cons_unique_not_tail_contains_light[A](head: A, tail: List[A]) {
    List.cons(head, tail).is_unique implies not tail.contains(head)
} by {
    if tail.contains(head) {
        list_contains_implies_count_geq_one[A](tail, head)
        tail.count(head) >= Nat.1
        List.cons(head, tail).count(head) = Nat.1 + tail.count(head)
        sum_lte(Nat.1, Nat.1, Nat.1, tail.count(head))
        Nat.1 + Nat.1 <= Nat.1 + tail.count(head)
        Nat.1 + Nat.1 = Nat.1.suc
        Nat.1.suc <= Nat.1 + tail.count(head)
        List.cons(head, tail).count(head) >= Nat.1.suc
        List.cons(head, tail).count(head) > Nat.1
        unique_implies_no_duplicate[A](List.cons(head, tail), head)
        List.cons(head, tail).count(head) <= Nat.1
        false
    }
}

lemma set_is_disjoint_union_of_disjoint[K](a: Set[K], b: Set[K], c: Set[K]) {
    a.is_disjoint(b) and a.is_disjoint(c) implies a.is_disjoint(b.union(c))
} by {
    forall(x: K) {
        if a.contains(x) and b.union(c).contains(x) {
            union_contains_cases(b, c, x)
            if b.contains(x) {
                a.contains(x) and b.contains(x)
                false
            }
            if c.contains(x) {
                a.contains(x) and c.contains(x)
                false
            }
            false
        }
    }
}

lemma set_disjoint_from_all_implies_disjoint_union_nil[K](s: Set[K]) {
    (forall(t: Set[K]) { List.nil[Set[K]].contains(t) implies s.is_disjoint(t) }) implies s.is_disjoint(set_list_union[K](List.nil[Set[K]]))
} by {
    set_list_union[K](List.nil[Set[K]]) = Set.empty_set[K]
    forall(x: K) {
        not (s.contains(x) and Set.empty_set[K].contains(x))
    }
    s.is_disjoint(Set.empty_set[K])
    s.is_disjoint(set_list_union[K](List.nil[Set[K]]))
}

/// A set disjoint from every set in a list is disjoint from their recursive union.
theorem set_disjoint_from_all_implies_disjoint_union[K](s: Set[K], items: List[Set[K]]) {
    (forall(t: Set[K]) { items.contains(t) implies s.is_disjoint(t) }) implies s.is_disjoint(set_list_union[K](items))
} by {
    let p: List[Set[K]] -> Bool = function(xs: List[Set[K]]) {
        (forall(t: Set[K]) { xs.contains(t) implies s.is_disjoint(t) }) implies s.is_disjoint(set_list_union[K](xs))
    }
    set_disjoint_from_all_implies_disjoint_union_nil[K](s)
    p(List.nil[Set[K]])
    forall(tail: List[Set[K]], head: Set[K]) {
        if p(tail) {
            p(tail) = ((forall(t: Set[K]) { tail.contains(t) implies s.is_disjoint(t) }) implies s.is_disjoint(set_list_union[K](tail)))
            if forall(t: Set[K]) { List.cons(head, tail).contains(t) implies s.is_disjoint(t) } {
                List.cons(head, tail).contains(head)
                s.is_disjoint(head)
                forall(t: Set[K]) {
                    if tail.contains(t) {
                        List.cons(head, tail).contains(t)
                        s.is_disjoint(t)
                    }
                }
                forall(t: Set[K]) { tail.contains(t) implies s.is_disjoint(t) }
                s.is_disjoint(set_list_union[K](tail))
                set_is_disjoint_union_of_disjoint[K](s, head, set_list_union[K](tail))
                s.is_disjoint(head.union(set_list_union[K](tail)))
                set_list_union[K](List.cons(head, tail)) = head.union(set_list_union[K](tail))
                s.is_disjoint(set_list_union[K](List.cons(head, tail)))
            }
            p(List.cons(head, tail))
        }
    }
    List.induction(p)
    p(items)
    p(items) = ((forall(t: Set[K]) { items.contains(t) implies s.is_disjoint(t) }) implies s.is_disjoint(set_list_union[K](items)))
}

lemma set_list_pairwise_disjoint_elim_curried[K](items: List[Set[K]], a: Set[K], b: Set[K]) {
    set_list_pairwise_disjoint[K](items) implies (items.contains(a) implies (items.contains(b) implies (a != b implies a.is_disjoint(b))))
} by {
    set_list_pairwise_disjoint[K](items) = forall(x: Set[K], y: Set[K]) { items.contains(x) and items.contains(y) and x != y implies x.is_disjoint(y) }
    forall(x: Set[K], y: Set[K]) { items.contains(x) and items.contains(y) and x != y implies x.is_disjoint(y) }
    if items.contains(a) {
        if items.contains(b) {
            if a != b {
                items.contains(a) and items.contains(b) and a != b
                a.is_disjoint(b)
            }
        }
    }
}

lemma set_list_pairwise_disjoint_tail_point[K](head: Set[K], tail: List[Set[K]], a: Set[K], b: Set[K]) {
    set_list_pairwise_disjoint[K](List.cons(head, tail)) and tail.contains(a) and tail.contains(b) and a != b implies a.is_disjoint(b)
} by {
    cons_contains_tail[Set[K]](head, tail, a)
    cons_contains_tail[Set[K]](head, tail, b)
    List.cons(head, tail).contains(a)
    List.cons(head, tail).contains(b)
    set_list_pairwise_disjoint_elim_curried[K](List.cons(head, tail), a, b)
}

/// Pairwise disjointness of a cons list descends to its tail.
theorem set_list_pairwise_disjoint_tail[K](head: Set[K], tail: List[Set[K]]) {
    set_list_pairwise_disjoint[K](List.cons(head, tail)) implies set_list_pairwise_disjoint[K](tail)
} by {
    forall(a: Set[K], b: Set[K]) {
        if tail.contains(a) and tail.contains(b) and a != b {
            set_list_pairwise_disjoint_tail_point[K](head, tail, a, b)
            a.is_disjoint(b)
        }
    }
}

lemma set_list_pairwise_disjoint_cons_head_tail_point[K](head: Set[K], tail: List[Set[K]], s: Set[K]) {
    set_list_pairwise_disjoint[K](List.cons(head, tail)) and not tail.contains(head) and tail.contains(s) implies head.is_disjoint(s)
} by {
    cons_contains_head[Set[K]](head, tail)
    cons_contains_tail[Set[K]](head, tail, s)
    List.cons(head, tail).contains(head)
    List.cons(head, tail).contains(s)
    if head = s {
        tail.contains(head)
        false
    }
    head != s
    set_list_pairwise_disjoint_elim_curried[K](List.cons(head, tail), head, s)
}

lemma set_list_pairwise_disjoint_cons_head_tail_all[K](head: Set[K], tail: List[Set[K]]) {
    set_list_pairwise_disjoint[K](List.cons(head, tail)) and not tail.contains(head) implies forall(s: Set[K]) { tail.contains(s) implies head.is_disjoint(s) }
} by {
    forall(s: Set[K]) {
        if tail.contains(s) {
            set_list_pairwise_disjoint_cons_head_tail_point[K](head, tail, s)
            head.is_disjoint(s)
        }
    }
}

lemma unique_pairwise_implies_recursively_disjoint_nil[K] {
    List.nil[Set[K]].is_unique and set_list_pairwise_disjoint[K](List.nil[Set[K]]) implies set_list_recursively_disjoint[K](List.nil[Set[K]])
} by {
    set_list_recursively_disjoint[K](List.nil[Set[K]]) = true
}

lemma unique_pairwise_implies_recursively_disjoint_cons_step[K](head: Set[K], tail: List[Set[K]]) {
    (tail.is_unique and set_list_pairwise_disjoint[K](tail) implies set_list_recursively_disjoint[K](tail)) and
    List.cons(head, tail).is_unique and set_list_pairwise_disjoint[K](List.cons(head, tail))
    implies set_list_recursively_disjoint[K](List.cons(head, tail))
} by {
    unique_implies_tail_unique[Set[K]](head, tail)
    tail.is_unique
    set_list_pairwise_disjoint_tail[K](head, tail)
    set_list_pairwise_disjoint[K](tail)
    set_list_recursively_disjoint[K](tail)
    cons_unique_not_tail_contains_light[Set[K]](head, tail)
    not tail.contains(head)
    set_list_pairwise_disjoint_cons_head_tail_all[K](head, tail)
    forall(s: Set[K]) { tail.contains(s) implies head.is_disjoint(s) }
    set_disjoint_from_all_implies_disjoint_union[K](head, tail)
    head.is_disjoint(set_list_union[K](tail))
    head.is_disjoint(set_list_union[K](tail)) and set_list_recursively_disjoint[K](tail)
    set_list_recursively_disjoint[K](List.cons(head, tail)) = (head.is_disjoint(set_list_union[K](tail)) and set_list_recursively_disjoint[K](tail))
    set_list_recursively_disjoint[K](List.cons(head, tail))
}

lemma unique_pairwise_recursive_statement_nil[K] {
    List.nil[Set[K]].is_unique and set_list_pairwise_disjoint[K](List.nil[Set[K]])
    implies set_list_recursively_disjoint[K](List.nil[Set[K]])
} by {
    unique_pairwise_implies_recursively_disjoint_nil[K]
}

lemma unique_pairwise_recursive_statement_cons_step[K](head: Set[K], tail: List[Set[K]]) {
    (tail.is_unique and set_list_pairwise_disjoint[K](tail) implies set_list_recursively_disjoint[K](tail))
    implies (List.cons(head, tail).is_unique and set_list_pairwise_disjoint[K](List.cons(head, tail))
    implies set_list_recursively_disjoint[K](List.cons(head, tail)))
} by {
    unique_pairwise_implies_recursively_disjoint_cons_step[K](head, tail)
}

/// A unique pairwise-disjoint set list is recursively disjoint.
theorem unique_pairwise_implies_recursively_disjoint[K](items: List[Set[K]]) {
    items.is_unique and set_list_pairwise_disjoint[K](items)
    implies set_list_recursively_disjoint[K](items)
} by {
    let p: List[Set[K]] -> Bool = function(xs: List[Set[K]]) {
        xs.is_unique and set_list_pairwise_disjoint[K](xs) implies set_list_recursively_disjoint[K](xs)
    }
    unique_pairwise_recursive_statement_nil[K]
    p(List.nil[Set[K]])
    forall(tail: List[Set[K]], head: Set[K]) {
        if p(tail) {
            p(tail) = (tail.is_unique and set_list_pairwise_disjoint[K](tail) implies set_list_recursively_disjoint[K](tail))
            unique_pairwise_recursive_statement_cons_step[K](head, tail)
            p(List.cons(head, tail))
        }
    }
    List.induction(p)
    p(items)
    p(items) = (items.is_unique and set_list_pairwise_disjoint[K](items) implies set_list_recursively_disjoint[K](items))
}

/// A finite/list partition is recursively disjoint in the order of its list.
theorem set_list_partition_recursively_disjoint[K](items: List[Set[K]], whole: Set[K]) {
    set_list_partition_of[K](items, whole) implies set_list_recursively_disjoint[K](items)
} by {
    if set_list_partition_of[K](items, whole) {
        set_list_partition_of[K](items, whole) =
            (set_list_exactly_covers[K](items, whole) and items.is_unique and set_list_pairwise_disjoint[K](items))
        unique_pairwise_implies_recursively_disjoint[K](items)
        set_list_recursively_disjoint[K](items)
    }
}

/// A unique pairwise-disjoint set list with common finite cardinality has union cardinality `n * length`.
theorem set_list_union_cardinality_is_of_unique_pairwise[K](items: List[Set[K]], n: Nat) {
    items.is_unique and set_list_pairwise_disjoint[K](items) and set_list_all_cardinality_is[K](items, n)
    implies set_list_union[K](items).cardinality_is(n * items.length)
} by {
    if items.is_unique and set_list_pairwise_disjoint[K](items) and set_list_all_cardinality_is[K](items, n) {
        unique_pairwise_implies_recursively_disjoint[K](items)
        set_list_recursively_disjoint[K](items)
        set_list_union_cardinality_recursive[K](items, n)
        set_list_union[K](items).cardinality_is(n * items.length)
    }
}

lemma set_list_cardinality_sum_nil[K](card: Set[K] -> Nat) {
    set_list_cardinality_sum[K](List.nil[Set[K]], card) = Nat.0
} by {
    map[Set[K], Nat](List.nil[Set[K]], card) = List.nil[Nat]
    sum[Nat](List.nil[Nat]) = Nat.0
}

lemma set_list_cardinality_sum_cons[K](head: Set[K], tail: List[Set[K]],
    card: Set[K] -> Nat) {
    set_list_cardinality_sum[K](List.cons(head, tail), card) =
    card(head) + set_list_cardinality_sum[K](tail, card)
} by {
    map[Set[K], Nat](List.cons(head, tail), card) =
        List.cons(card(head), map[Set[K], Nat](tail, card))
    sum[Nat](List.cons(card(head), map[Set[K], Nat](tail, card))) =
        card(head) + sum[Nat](map[Set[K], Nat](tail, card))
}

lemma set_list_union_cardinality_at_most_sum_nil[K](card: Set[K] -> Nat) {
    set_list_union[K](List.nil[Set[K]]).cardinality_at_most(
        set_list_cardinality_sum[K](List.nil[Set[K]], card))
} by {
    set_list_union[K](List.nil[Set[K]]) = Set.empty_set[K]
    Set.empty_set[K].cardinality_is(Nat.0)
    cardinality_is_smallest_cardinality[K](Set.empty_set[K], Nat.0)
    Set.empty_set[K].cardinality_at_most(Nat.0)
    set_list_cardinality_sum_nil[K](card)
}

lemma set_list_union_cardinality_at_most_sum_cons[K](head: Set[K],
    tail: List[Set[K]], card: Set[K] -> Nat) {
    head.cardinality_is(card(head)) and
    set_list_union[K](tail).cardinality_at_most(
        set_list_cardinality_sum[K](tail, card))
    implies set_list_union[K](List.cons(head, tail)).cardinality_at_most(
        set_list_cardinality_sum[K](List.cons(head, tail), card))
} by {
    if head.cardinality_is(card(head)) and
        set_list_union[K](tail).cardinality_at_most(
            set_list_cardinality_sum[K](tail, card)) {
        cardinality_is_smallest_cardinality[K](head, card(head))
        head.cardinality_at_most(card(head))
        union_is_at_most_length[K](head, set_list_union[K](tail), card(head),
            set_list_cardinality_sum[K](tail, card))
        head.union(set_list_union[K](tail)).cardinality_at_most(
            card(head) + set_list_cardinality_sum[K](tail, card))
        set_list_cardinality_sum_cons[K](head, tail, card)
        set_list_union[K](List.cons(head, tail)) = head.union(set_list_union[K](tail))
        set_list_union[K](List.cons(head, tail)).cardinality_at_most(
            set_list_cardinality_sum[K](List.cons(head, tail), card))
    }
}

/// The union of a finite list of sets has cardinality at most the sum of their cardinalities.
theorem set_list_union_cardinality_at_most_sum[K](items: List[Set[K]],
    card: Set[K] -> Nat) {
    set_list_all_cardinality_by[K](items, card)
    implies set_list_union[K](items).cardinality_at_most(
        set_list_cardinality_sum[K](items, card))
} by {
    define p(xs: List[Set[K]]) -> Bool {
        set_list_all_cardinality_by[K](xs, card)
        implies set_list_union[K](xs).cardinality_at_most(
            set_list_cardinality_sum[K](xs, card))
    }
    set_list_union_cardinality_at_most_sum_nil[K](card)
    p(List.nil[Set[K]])
    forall(head: Set[K], tail: List[Set[K]]) {
        if p(tail) {
            if set_list_all_cardinality_by[K](List.cons(head, tail), card) {
                set_list_all_cardinality_by_head[K](head, tail, card)
                head.cardinality_is(card(head))
                set_list_all_cardinality_by_tail[K](head, tail, card)
                set_list_all_cardinality_by[K](tail, card)
                set_list_union[K](tail).cardinality_at_most(
                    set_list_cardinality_sum[K](tail, card))
                set_list_union_cardinality_at_most_sum_cons[K](head, tail, card)
                set_list_union[K](List.cons(head, tail)).cardinality_at_most(
                    set_list_cardinality_sum[K](List.cons(head, tail), card))
            }
            p(List.cons(head, tail)) =
                (set_list_all_cardinality_by[K](List.cons(head, tail), card)
                implies set_list_union[K](List.cons(head, tail)).cardinality_at_most(
                    set_list_cardinality_sum[K](List.cons(head, tail), card)))
            p(List.cons(head, tail))
        }
    }
    List.induction(p)
    p(items)
}

lemma set_list_union_cardinality_by_recursive_cons_step[K](head: Set[K],
    tail: List[Set[K]], card: Set[K] -> Nat) {
    head.cardinality_is(card(head)) and
    set_list_union[K](tail).cardinality_is(set_list_cardinality_sum[K](tail, card)) and
    set_list_recursively_disjoint[K](List.cons(head, tail))
    implies set_list_union[K](List.cons(head, tail)).cardinality_is(
        set_list_cardinality_sum[K](List.cons(head, tail), card))
} by {
    head.is_disjoint(set_list_union[K](tail))
    head.cardinality_is(card(head)) and
        set_list_union[K](tail).cardinality_is(set_list_cardinality_sum[K](tail, card)) and
        head.is_disjoint(set_list_union[K](tail))
    disjoint_union_is_length[K](head, set_list_union[K](tail), card(head),
        set_list_cardinality_sum[K](tail, card))
    head.union(set_list_union[K](tail)).cardinality_is(
        card(head) + set_list_cardinality_sum[K](tail, card))
    set_list_cardinality_sum_cons[K](head, tail, card)
    card(head) + set_list_cardinality_sum[K](tail, card) =
        set_list_cardinality_sum[K](List.cons(head, tail), card)
    head.union(set_list_union[K](tail)).cardinality_is(
        set_list_cardinality_sum[K](List.cons(head, tail), card))
    set_list_union[K](List.cons(head, tail)) = head.union(set_list_union[K](tail))
}

lemma set_list_union_cardinality_by_recursive_nil[K](card: Set[K] -> Nat) {
    set_list_all_cardinality_by[K](List.nil[Set[K]], card) and
    set_list_recursively_disjoint[K](List.nil[Set[K]])
    implies set_list_union[K](List.nil[Set[K]]).cardinality_is(
        set_list_cardinality_sum[K](List.nil[Set[K]], card))
} by {
    set_list_union[K](List.nil[Set[K]]) = Set.empty_set[K]
    Set.empty_set[K].cardinality_is(Nat.0)
    set_list_cardinality_sum_nil[K](card)
    set_list_cardinality_sum[K](List.nil[Set[K]], card) = Nat.0
}

/// A recursively disjoint list of sets has union cardinality equal to the sum of prescribed member cardinalities.
theorem set_list_union_cardinality_by_recursive[K](items: List[Set[K]], card: Set[K] -> Nat) {
    set_list_all_cardinality_by[K](items, card) and
    set_list_recursively_disjoint[K](items)
    implies set_list_union[K](items).cardinality_is(set_list_cardinality_sum[K](items, card))
} by {
    define p(xs: List[Set[K]]) -> Bool {
        set_list_all_cardinality_by[K](xs, card) and
        set_list_recursively_disjoint[K](xs)
        implies set_list_union[K](xs).cardinality_is(set_list_cardinality_sum[K](xs, card))
    }
    set_list_union_cardinality_by_recursive_nil[K](card)
    p(List.nil[Set[K]])
    forall(head: Set[K], tail: List[Set[K]]) {
        if p(tail) {
            if set_list_all_cardinality_by[K](List.cons(head, tail), card) and
                set_list_recursively_disjoint[K](List.cons(head, tail)) {
                set_list_all_cardinality_by_head[K](head, tail, card)
                head.cardinality_is(card(head))
                set_list_all_cardinality_by_tail[K](head, tail, card)
                set_list_all_cardinality_by[K](tail, card)
                set_list_recursively_disjoint_tail[K](head, tail)
                set_list_recursively_disjoint[K](tail)
                p(tail) = (set_list_all_cardinality_by[K](tail, card) and
                    set_list_recursively_disjoint[K](tail)
                    implies set_list_union[K](tail).cardinality_is(
                        set_list_cardinality_sum[K](tail, card)))
                set_list_union[K](tail).cardinality_is(set_list_cardinality_sum[K](tail, card))
                set_list_union_cardinality_by_recursive_cons_step[K](head, tail, card)
                set_list_union[K](List.cons(head, tail)).cardinality_is(
                    set_list_cardinality_sum[K](List.cons(head, tail), card))
            }
            p(List.cons(head, tail))
        }
    }
    List.induction(function(xs: List[Set[K]]) { p(xs) })
    forall(xs: List[Set[K]]) { p(xs) }
    p(items)
}

/// A unique pairwise-disjoint list of sets has union cardinality equal to the sum of prescribed member cardinalities.
theorem set_list_union_cardinality_by_of_unique_pairwise[K](items: List[Set[K]],
    card: Set[K] -> Nat) {
    items.is_unique and set_list_pairwise_disjoint[K](items) and
    set_list_all_cardinality_by[K](items, card)
    implies set_list_union[K](items).cardinality_is(set_list_cardinality_sum[K](items, card))
} by {
    if items.is_unique and set_list_pairwise_disjoint[K](items) and
        set_list_all_cardinality_by[K](items, card) {
        unique_pairwise_implies_recursively_disjoint[K](items)
        set_list_recursively_disjoint[K](items)
        set_list_union_cardinality_by_recursive[K](items, card)
        set_list_union[K](items).cardinality_is(set_list_cardinality_sum[K](items, card))
    }
}

/// A finite/list partition has cardinality equal to the sum of its member cardinalities.
theorem set_list_partition_cardinality_by[K](items: List[Set[K]], whole: Set[K],
    card: Set[K] -> Nat) {
    set_list_partition_of[K](items, whole) and set_list_all_cardinality_by[K](items, card)
    implies whole.cardinality_is(set_list_cardinality_sum[K](items, card))
} by {
    if set_list_partition_of[K](items, whole) and set_list_all_cardinality_by[K](items, card) {
        set_list_partition_of[K](items, whole) =
            (set_list_exactly_covers[K](items, whole) and items.is_unique and set_list_pairwise_disjoint[K](items))
        set_list_union_cardinality_by_of_unique_pairwise[K](items, card)
        set_list_union[K](items).cardinality_is(set_list_cardinality_sum[K](items, card))
        set_list_exactly_covers[K](items, whole) = (set_list_union[K](items) = whole)
        set_list_union[K](items) = whole
        whole.cardinality_is(set_list_cardinality_sum[K](items, card))
    }
}

/// A finite/list partition with common member cardinality has cardinality `n * length`.
theorem set_list_partition_cardinality_is[K](items: List[Set[K]], whole: Set[K], n: Nat) {
    set_list_partition_of[K](items, whole) and set_list_all_cardinality_is[K](items, n)
    implies whole.cardinality_is(n * items.length)
} by {
    if set_list_partition_of[K](items, whole) and set_list_all_cardinality_is[K](items, n) {
        set_list_partition_of[K](items, whole) =
            (set_list_exactly_covers[K](items, whole) and items.is_unique and set_list_pairwise_disjoint[K](items))
        set_list_union_cardinality_is_of_unique_pairwise[K](items, n)
        set_list_union[K](items).cardinality_is(n * items.length)
        set_list_exactly_covers[K](items, whole) = (set_list_union[K](items) = whole)
        set_list_union[K](items) = whole
        whole.cardinality_is(n * items.length)
    }
}

lemma set_list_union_contains_eq_exists_member_nil[K](x: K) {
    set_list_union[K](List.nil[Set[K]]).contains(x) = exists(c: Set[K]) {
        List.nil[Set[K]].contains(c) and c.contains(x)
    }
} by {
    set_list_union[K](List.nil[Set[K]]) = Set.empty_set[K]
    empty_set_contains_eq[K](x)
    set_list_union[K](List.nil[Set[K]]).contains(x) = false
    if exists(c: Set[K]) {
        List.nil[Set[K]].contains(c) and c.contains(x)
    } {
        let c: Set[K] satisfy {
            List.nil[Set[K]].contains(c) and c.contains(x)
        }
        false
    }
    not exists(c: Set[K]) {
        List.nil[Set[K]].contains(c) and c.contains(x)
    }
}

lemma set_list_union_contains_eq_exists_member_cons_step[K](head: Set[K],
    tail: List[Set[K]], x: K) {
    (set_list_union[K](tail).contains(x) = exists(c: Set[K]) {
        tail.contains(c) and c.contains(x)
    }) implies (set_list_union[K](List.cons(head, tail)).contains(x) = exists(c: Set[K]) {
        List.cons(head, tail).contains(c) and c.contains(x)
    })
} by {
    if set_list_union[K](tail).contains(x) = exists(c: Set[K]) {
        tail.contains(c) and c.contains(x)
    } {
        if set_list_union[K](List.cons(head, tail)).contains(x) {
            set_list_union[K](List.cons(head, tail)) = head.union(set_list_union[K](tail))
            head.union(set_list_union[K](tail)).contains(x)
            union_contains_cases(head, set_list_union[K](tail), x)
            if head.contains(x) {
                List.cons(head, tail).contains(head)
                exists(c: Set[K]) {
                    c = head and List.cons(head, tail).contains(c) and c.contains(x)
                }
            }
            if set_list_union[K](tail).contains(x) {
                exists(c: Set[K]) {
                    tail.contains(c) and c.contains(x)
                }
                let c: Set[K] satisfy {
                    tail.contains(c) and c.contains(x)
                }
                List.cons(head, tail).contains(c)
                exists(d: Set[K]) {
                    d = c and List.cons(head, tail).contains(d) and d.contains(x)
                }
            }
            if not exists(c: Set[K]) {
                List.cons(head, tail).contains(c) and c.contains(x)
            } {
                if head.contains(x) {
                    exists(c: Set[K]) {
                        c = head and List.cons(head, tail).contains(c) and c.contains(x)
                    }
                    false
                }
                if set_list_union[K](tail).contains(x) {
                    exists(c: Set[K]) {
                        tail.contains(c) and c.contains(x)
                    }
                    let c: Set[K] satisfy {
                        tail.contains(c) and c.contains(x)
                    }
                    List.cons(head, tail).contains(c)
                    exists(d: Set[K]) {
                        d = c and List.cons(head, tail).contains(d) and d.contains(x)
                    }
                    false
                }
                union_contains_cases(head, set_list_union[K](tail), x)
                false
            }
            exists(c: Set[K]) {
                List.cons(head, tail).contains(c) and c.contains(x)
            }
        }
        if exists(c: Set[K]) {
            List.cons(head, tail).contains(c) and c.contains(x)
        } {
            let c: Set[K] satisfy {
                List.cons(head, tail).contains(c) and c.contains(x)
            }
            set_list_union_contains_of_member[K](List.cons(head, tail), c, x)
            set_list_union[K](List.cons(head, tail)).contains(x)
        }
        set_list_union[K](List.cons(head, tail)).contains(x) = exists(c: Set[K]) {
            List.cons(head, tail).contains(c) and c.contains(x)
        }
    }
}

/// Membership in the recursive list union is equivalent to membership in some listed set.
theorem set_list_union_contains_eq_exists_member[K](items: List[Set[K]], x: K) {
    set_list_union[K](items).contains(x) = exists(c: Set[K]) {
        items.contains(c) and c.contains(x)
    }
} by {
    let p: List[Set[K]] -> Bool = function(xs: List[Set[K]]) {
        set_list_union[K](xs).contains(x) = exists(c: Set[K]) {
            xs.contains(c) and c.contains(x)
        }
    }
    set_list_union_contains_eq_exists_member_nil[K](x)
    p(List.nil[Set[K]])
    forall(head: Set[K], tail: List[Set[K]]) {
        if p(tail) {
            set_list_union_contains_eq_exists_member_cons_step[K](head, tail, x)
            p(List.cons(head, tail))
        }
    }
    List.induction(function(xs: List[Set[K]]) { p(xs) })
    forall(xs: List[Set[K]]) { p(xs) }
    p(items)
    p(items) = (set_list_union[K](items).contains(x) = exists(c: Set[K]) {
        items.contains(c) and c.contains(x)
    })
}

/// Membership in the recursive list union has a witnessing member set from the list.
theorem set_list_union_contains_witness[K](items: List[Set[K]], x: K) {
    set_list_union[K](items).contains(x) implies exists(c: Set[K]) {
        items.contains(c) and c.contains(x)
    }
} by {
    set_list_union_contains_eq_exists_member[K](items, x)
    if set_list_union[K](items).contains(x) {
        exists(c: Set[K]) {
            items.contains(c) and c.contains(x)
        }
    }
}

/// In a finite/list partition, every member set is contained in the covered set.
theorem set_list_partition_member_subset[K](items: List[Set[K]], whole: Set[K], c: Set[K]) {
    set_list_partition_of[K](items, whole) and items.contains(c) implies c.subset(whole)
} by {
    if set_list_partition_of[K](items, whole) and items.contains(c) {
        set_list_partition_of[K](items, whole) =
            (set_list_exactly_covers[K](items, whole) and items.is_unique and set_list_pairwise_disjoint[K](items))
        set_list_exactly_covers[K](items, whole) = (set_list_union[K](items) = whole)
        set_list_union[K](items) = whole
        forall(x: K) {
            if c.contains(x) {
                set_list_union_contains_of_member[K](items, c, x)
                set_list_union[K](items).contains(x)
                whole.contains(x)
            }
        }
    }
}

/// Membership in the covered set of a finite/list partition has a witnessing member set.
theorem set_list_partition_contains_witness[K](items: List[Set[K]], whole: Set[K], x: K) {
    set_list_partition_of[K](items, whole) and whole.contains(x) implies exists(c: Set[K]) {
        items.contains(c) and c.contains(x)
    }
} by {
    if set_list_partition_of[K](items, whole) and whole.contains(x) {
        set_list_partition_of[K](items, whole) =
            (set_list_exactly_covers[K](items, whole) and items.is_unique and set_list_pairwise_disjoint[K](items))
        set_list_exactly_covers[K](items, whole) = (set_list_union[K](items) = whole)
        set_list_union[K](items) = whole
        set_list_union[K](items).contains(x)
        set_list_union_contains_witness[K](items, x)
        exists(c: Set[K]) {
            items.contains(c) and c.contains(x)
        }
    }
}

/// The recursive list union is the list-indexed union of the identity family.
theorem set_list_union_eq_list_indexed_union_self[K](items: List[Set[K]]) {
    set_list_union[K](items) = list_indexed_union[K, Set[K]](items, function(s: Set[K]) { s })
} by {
    let u = set_list_union[K](items)
    let v = list_indexed_union[K, Set[K]](items, function(s: Set[K]) { s })
    forall(x: K) {
        if u.contains(x) {
            set_list_union_contains_witness[K](items, x)
            let c: Set[K] satisfy {
                items.contains(c) and c.contains(x)
            }
            list_indexed_union_contains_of_contains[K, Set[K]](items, function(s: Set[K]) { s }, c, x)
            v.contains(x)
        }
        if v.contains(x) {
            list_indexed_union_contains_witness[K, Set[K]](items, function(s: Set[K]) { s }, x)
            let c: Set[K] satisfy {
                items.contains(c) and function(s: Set[K]) { s }(c).contains(x)
            }
            c.contains(x)
            set_list_union_contains_of_member[K](items, c, x)
            u.contains(x)
        }
        u.contains(x) = v.contains(x)
    }
    set_ext(u, v)
}

/// The list-indexed union of the identity family has cardinality `n * length` for a unique pairwise-disjoint list with common cardinality `n`.
theorem list_indexed_union_self_cardinality_is_of_unique_pairwise[K](items: List[Set[K]],
    n: Nat) {
    items.is_unique and set_list_pairwise_disjoint[K](items) and set_list_all_cardinality_is[K](items, n)
    implies list_indexed_union[K, Set[K]](items, function(s: Set[K]) { s }).cardinality_is(n * items.length)
} by {
    if items.is_unique and set_list_pairwise_disjoint[K](items) and set_list_all_cardinality_is[K](items, n) {
        set_list_union_cardinality_is_of_unique_pairwise[K](items, n)
        set_list_union[K](items).cardinality_is(n * items.length)
        set_list_union_eq_list_indexed_union_self[K](items)
        list_indexed_union[K, Set[K]](items, function(s: Set[K]) { s }).cardinality_is(n * items.length)
    }
}

/// The list-indexed union of the identity family has cardinality equal to the sum of prescribed cardinalities for a unique pairwise-disjoint list.
theorem list_indexed_union_self_cardinality_by_of_unique_pairwise[K](items: List[Set[K]],
    card: Set[K] -> Nat) {
    items.is_unique and set_list_pairwise_disjoint[K](items) and
    set_list_all_cardinality_by[K](items, card)
    implies list_indexed_union[K, Set[K]](items, function(s: Set[K]) { s }).cardinality_is(
        set_list_cardinality_sum[K](items, card))
} by {
    if items.is_unique and set_list_pairwise_disjoint[K](items) and
        set_list_all_cardinality_by[K](items, card) {
        set_list_union_cardinality_by_of_unique_pairwise[K](items, card)
        set_list_union[K](items).cardinality_is(set_list_cardinality_sum[K](items, card))
        set_list_union_eq_list_indexed_union_self[K](items)
        list_indexed_union[K, Set[K]](items, function(s: Set[K]) { s }).cardinality_is(
            set_list_cardinality_sum[K](items, card))
    }
}

/// Recursive union after mapping a set-valued family over a list equals the corresponding list-indexed union.
theorem set_list_union_map_eq_list_indexed_union[K, I](items: List[I], family: I -> Set[K]) {
    set_list_union[K](map[I, Set[K]](items, family)) = list_indexed_union[K, I](items, family)
} by {
    let u = set_list_union[K](map[I, Set[K]](items, family))
    let v = list_indexed_union[K, I](items, family)
    forall(x: K) {
        if u.contains(x) {
            set_list_union_contains_witness[K](map[I, Set[K]](items, family), x)
            let c: Set[K] satisfy {
                map[I, Set[K]](items, family).contains(c) and c.contains(x)
            }
            map_contains[I, Set[K]](items, family, c)
            let i: I satisfy {
                items.contains(i) and family(i) = c
            }
            family(i).contains(x)
            list_indexed_union_contains_of_contains[K, I](items, family, i, x)
            v.contains(x)
        }
        if v.contains(x) {
            list_indexed_union_contains_witness[K, I](items, family, x)
            let i: I satisfy {
                items.contains(i) and family(i).contains(x)
            }
            map_contains_of_contains[I, Set[K]](items, family, i)
            map[I, Set[K]](items, family).contains(family(i))
            set_list_union_contains_of_member[K](map[I, Set[K]](items, family), family(i), x)
            u.contains(x)
        }
        u.contains(x) = v.contains(x)
    }
    set_ext(u, v)
}

lemma set_list_all_cardinality_by_map_point[K, I](items: List[I], family: I -> Set[K],
    card: Set[K] -> Nat, s: Set[K]) {
    (forall(i: I) { items.contains(i) implies family(i).cardinality_is(card(family(i))) }) and
    map[I, Set[K]](items, family).contains(s) implies s.cardinality_is(card(s))
} by {
    if forall(i: I) { items.contains(i) implies family(i).cardinality_is(card(family(i))) } {
        if map[I, Set[K]](items, family).contains(s) {
            map_contains[I, Set[K]](items, family, s)
            let i: I satisfy {
                items.contains(i) and family(i) = s
            }
            items.contains(i)
            items.contains(i) implies family(i).cardinality_is(card(family(i)))
            family(i).cardinality_is(card(family(i)))
            s.cardinality_is(card(s))
        }
    }
}

/// Pointwise cardinality data for a list-indexed set family gives the mapped set-list cardinality predicate.
theorem set_list_all_cardinality_by_map[K, I](items: List[I], family: I -> Set[K],
    card: Set[K] -> Nat) {
    (forall(i: I) { items.contains(i) implies family(i).cardinality_is(card(family(i))) })
    implies set_list_all_cardinality_by[K](map[I, Set[K]](items, family), card)
} by {
    if forall(i: I) { items.contains(i) implies family(i).cardinality_is(card(family(i))) } {
        forall(s: Set[K]) {
            if map[I, Set[K]](items, family).contains(s) {
                set_list_all_cardinality_by_map_point[K, I](items, family, card, s)
                s.cardinality_is(card(s))
            }
        }
        set_list_all_cardinality_by[K](map[I, Set[K]](items, family), card) = forall(s: Set[K]) {
            map[I, Set[K]](items, family).contains(s) implies s.cardinality_is(card(s))
        }
        set_list_all_cardinality_by[K](map[I, Set[K]](items, family), card)
    }
}

lemma set_list_all_cardinality_is_map_point[K, I](items: List[I], family: I -> Set[K],
    n: Nat, s: Set[K]) {
    (forall(i: I) { items.contains(i) implies family(i).cardinality_is(n) }) and
    map[I, Set[K]](items, family).contains(s) implies s.cardinality_is(n)
} by {
    if forall(i: I) { items.contains(i) implies family(i).cardinality_is(n) } {
        if map[I, Set[K]](items, family).contains(s) {
            map_contains[I, Set[K]](items, family, s)
            let i: I satisfy {
                items.contains(i) and family(i) = s
            }
            items.contains(i)
            items.contains(i) implies family(i).cardinality_is(n)
            family(i).cardinality_is(n)
            s.cardinality_is(n)
        }
    }
}

/// Pointwise common cardinality data for a list-indexed set family gives the mapped set-list common-cardinality predicate.
theorem set_list_all_cardinality_is_map[K, I](items: List[I], family: I -> Set[K],
    n: Nat) {
    (forall(i: I) { items.contains(i) implies family(i).cardinality_is(n) })
    implies set_list_all_cardinality_is[K](map[I, Set[K]](items, family), n)
} by {
    if forall(i: I) { items.contains(i) implies family(i).cardinality_is(n) } {
        forall(s: Set[K]) {
            if map[I, Set[K]](items, family).contains(s) {
                set_list_all_cardinality_is_map_point[K, I](items, family, n, s)
                s.cardinality_is(n)
            }
        }
        set_list_all_cardinality_is[K](map[I, Set[K]](items, family), n) = forall(s: Set[K]) {
            map[I, Set[K]](items, family).contains(s) implies s.cardinality_is(n)
        }
        set_list_all_cardinality_is[K](map[I, Set[K]](items, family), n)
    }
}

/// A list-indexed family has the cardinality sum of its mapped member list when the mapped family is unique and pairwise disjoint.
theorem list_indexed_union_cardinality_by_of_unique_pairwise_map[K, I](items: List[I],
    family: I -> Set[K], card: Set[K] -> Nat) {
    map[I, Set[K]](items, family).is_unique and
    set_list_pairwise_disjoint[K](map[I, Set[K]](items, family)) and
    (forall(i: I) { items.contains(i) implies family(i).cardinality_is(card(family(i))) })
    implies list_indexed_union[K, I](items, family).cardinality_is(
        set_list_cardinality_sum[K](map[I, Set[K]](items, family), card))
} by {
    if map[I, Set[K]](items, family).is_unique and
        set_list_pairwise_disjoint[K](map[I, Set[K]](items, family)) and
        forall(i: I) { items.contains(i) implies family(i).cardinality_is(card(family(i))) } {
        set_list_all_cardinality_by_map[K, I](items, family, card)
        set_list_all_cardinality_by[K](map[I, Set[K]](items, family), card)
        set_list_union_cardinality_by_of_unique_pairwise[K](map[I, Set[K]](items, family), card)
        set_list_union[K](map[I, Set[K]](items, family)).cardinality_is(
            set_list_cardinality_sum[K](map[I, Set[K]](items, family), card))
        set_list_union_map_eq_list_indexed_union[K, I](items, family)
        list_indexed_union[K, I](items, family).cardinality_is(
            set_list_cardinality_sum[K](map[I, Set[K]](items, family), card))
    }
}
