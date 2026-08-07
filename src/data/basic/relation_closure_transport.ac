/// Transport lemmas connecting relation closures with pushforward/pullback and set image/preimage.

from nat import Nat, alt_induction, zero_or_suc
from data.basic.functions import is_injective_fn, is_surjective_fn, is_bijection_fn,
    bijection_fn_is_injective, bijection_fn_is_surjective, injective_fn_eq,
    surjective_fn_has_preimage
from data.basic.relation_basic import eq_relation, relation_subset, relation_subset_step,
    relation_compose, relation_compose_intro, relation_compose_has_witness
from data.basic.relation import relation_power, relation_power_zero, relation_power_suc,
    relation_power_zero_self, relation_power_one, relation_power_one_step,
    relation_power_in_refl_trans_closure, relation_power_positive_in_trans_closure,
    relation_refl_trans_closure, relation_trans_closure
from data.basic.relation_transport import relation_pullback, relation_pushforward, relation_pushforward_intro,
    relation_pushforward_has_preimages
from data.basic.set import set_ext, subset_contains_eq, double_inclusion, relation_support,
    relation_support_contains_witness, relation_support_contains_of_related_left,
    equivalence_class, equivalence_class_contains_eq, set_image, maps_into_set_image,
    set_image_contains_witness, set_preimage, set_preimage_contains_eq

/// One successor step for pushing finite relation powers forward.
theorem relation_power_pushforward_subset_step[A, B](f: A -> B, r: (A, A) -> Bool, n: Nat) {
    relation_subset(
        relation_pushforward(f, relation_power(r, n)),
        relation_power(relation_pushforward(f, r), n)
    )
    implies
    relation_subset(
        relation_pushforward(f, relation_power(r, n.suc)),
        relation_power(relation_pushforward(f, r), n.suc)
    )
} by {
    if relation_subset(
        relation_pushforward(f, relation_power(r, n)),
        relation_power(relation_pushforward(f, r), n)
    ) {
        forall(u: B, v: B) {
            if relation_pushforward(f, relation_power(r, n.suc), u, v) {
                relation_pushforward_has_preimages(f, relation_power(r, n.suc), u, v)
                let x: A satisfy {
                    exists(y0: A) {
                        f(x) = u and f(y0) = v and relation_power(r, n.suc, x, y0)
                    }
                }
                let y: A satisfy {
                    f(x) = u and f(y) = v and relation_power(r, n.suc, x, y)
                }
                relation_power_suc(r, n)
                relation_compose_has_witness(relation_power(r, n), r, x, y)
                let z: A satisfy {
                    relation_power(r, n, x, z) and r(z, y)
                }
                exists(a: A, b: A) {
                    a = x and b = z and f(a) = f(x) and f(b) = f(z) and relation_power(r, n, a, b)
                }
                relation_pushforward(f, relation_power(r, n), f(x), f(z))
                relation_subset_step(
                    relation_pushforward(f, relation_power(r, n)),
                    relation_power(relation_pushforward(f, r), n),
                    f(x),
                    f(z)
                )
                relation_power(relation_pushforward(f, r), n, f(x), f(z))
                relation_pushforward_intro(f, r, z, y)
                relation_pushforward(f, r, f(z), f(y))
                relation_compose_intro(
                    relation_power(relation_pushforward(f, r), n),
                    relation_pushforward(f, r),
                    f(x),
                    f(z),
                    f(y)
                )
                relation_compose(
                    relation_power(relation_pushforward(f, r), n),
                    relation_pushforward(f, r),
                    f(x),
                    f(y)
                )
                relation_power_suc(relation_pushforward(f, r), n)
                relation_power(relation_pushforward(f, r), n.suc, f(x), f(y))
                relation_power(relation_pushforward(f, r), n.suc, u, v)
            }
        }
    }
}

/// Pushing forward a finite relation power lands in the corresponding power of the pushed-forward relation.
theorem relation_power_pushforward_subset[A, B](f: A -> B, r: (A, A) -> Bool, n: Nat) {
    relation_subset(
        relation_pushforward(f, relation_power(r, n)),
        relation_power(relation_pushforward(f, r), n)
    )
} by {
    define p(k: Nat) -> Bool {
        relation_subset(
            relation_pushforward(f, relation_power(r, k)),
            relation_power(relation_pushforward(f, r), k)
        )
    }
    forall(u: B, v: B) {
        if relation_pushforward(f, relation_power(r, Nat.zero), u, v) {
            relation_pushforward_has_preimages(f, relation_power(r, Nat.zero), u, v)
            let x: A satisfy {
                exists(y0: A) {
                    f(x) = u and f(y0) = v and relation_power(r, Nat.zero, x, y0)
                }
            }
            let y: A satisfy {
                f(x) = u and f(y) = v and relation_power(r, Nat.zero, x, y)
            }
            relation_power_zero(r)
            eq_relation[A](x, y)
            x = y
            f(x) = f(y)
            u = v
            eq_relation[B](u, v)
            relation_power_zero(relation_pushforward(f, r))
            relation_power(relation_pushforward(f, r), Nat.zero, u, v)
        }
    }
    relation_subset(
        relation_pushforward(f, relation_power(r, Nat.zero)),
        relation_power(relation_pushforward(f, r), Nat.zero)
    )
    p(Nat.zero)
    forall(k: Nat) {
        if p(k) {
            relation_power_pushforward_subset_step(f, r, k)
            p(k.suc)
        }
    }
    p(n)
}

/// The first positive pushed-forward power reflects back to a pushed-forward first power.
theorem relation_power_positive_pushforward_reverse_base[A, B](f: A -> B, r: (A, A) -> Bool) {
    relation_subset(
        relation_power(relation_pushforward(f, r), Nat.zero.suc),
        relation_pushforward(f, relation_power(r, Nat.zero.suc))
    )
} by {
    forall(u: B, v: B) {
        if relation_power(relation_pushforward(f, r), Nat.zero.suc, u, v) {
            relation_power_one(relation_pushforward(f, r))
            relation_pushforward(f, r, u, v)
            relation_pushforward_has_preimages(f, r, u, v)
            let x: A satisfy {
                exists(y0: A) {
                    f(x) = u and f(y0) = v and r(x, y0)
                }
            }
            let y: A satisfy {
                f(x) = u and f(y) = v and r(x, y)
            }
            relation_power_one_step(r, x, y)
            relation_power(r, Nat.zero.suc, x, y)
            exists(a: A, b: A) {
                a = x and b = y and f(a) = u and f(b) = v and
                relation_power(r, Nat.zero.suc, a, b)
            }
            relation_pushforward(f, relation_power(r, Nat.zero.suc), u, v)
        }
    }
}

/// One successor-step instance for reflecting positive powers of a pushed-forward relation.
theorem relation_power_positive_pushforward_reverse_step_of_injective_at[A, B](
    f: A -> B,
    r: (A, A) -> Bool,
    n: Nat,
    u: B,
    v: B
) {
    is_injective_fn(f) and
    relation_subset(
        relation_power(relation_pushforward(f, r), n.suc),
        relation_pushforward(f, relation_power(r, n.suc))
    ) and
    relation_power(relation_pushforward(f, r), n.suc.suc, u, v)
    implies
    relation_pushforward(f, relation_power(r, n.suc.suc), u, v)
} by {
    if is_injective_fn(f) and
        relation_subset(
            relation_power(relation_pushforward(f, r), n.suc),
            relation_pushforward(f, relation_power(r, n.suc))
        ) and
        relation_power(relation_pushforward(f, r), n.suc.suc, u, v) {
        relation_power_suc(relation_pushforward(f, r), n.suc)
        relation_compose_has_witness(
            relation_power(relation_pushforward(f, r), n.suc),
            relation_pushforward(f, r),
            u,
            v
        )
        let w: B satisfy {
            relation_power(relation_pushforward(f, r), n.suc, u, w) and
            relation_pushforward(f, r, w, v)
        }
        relation_subset_step(
            relation_power(relation_pushforward(f, r), n.suc),
            relation_pushforward(f, relation_power(r, n.suc)),
            u,
            w
        )
        relation_pushforward(f, relation_power(r, n.suc), u, w)
        relation_pushforward_has_preimages(f, relation_power(r, n.suc), u, w)
        let x: A satisfy {
            exists(z0: A) {
                f(x) = u and f(z0) = w and relation_power(r, n.suc, x, z0)
            }
        }
        let z: A satisfy {
            f(x) = u and f(z) = w and relation_power(r, n.suc, x, z)
        }
        relation_pushforward_has_preimages(f, r, w, v)
        let a: A satisfy {
            exists(y0: A) {
                f(a) = w and f(y0) = v and r(a, y0)
            }
        }
        let y: A satisfy {
            f(a) = w and f(y) = v and r(a, y)
        }
        f(z) = w
        f(a) = w
        f(z) = f(a)
        injective_fn_eq(f, z, a)
        z = a
        r(z, y)
        relation_compose_intro(relation_power(r, n.suc), r, x, z, y)
        relation_compose(relation_power(r, n.suc), r, x, y)
        relation_power_suc(r, n.suc)
        relation_power(r, n.suc.suc, x, y)
        exists(x0: A, y0: A) {
            x0 = x and y0 = y and f(x0) = u and f(y0) = v and
            relation_power(r, n.suc.suc, x0, y0)
        }
        relation_pushforward(f, relation_power(r, n.suc.suc), u, v)
    }
}


/// Injective maps reflect every positive power of a pushed-forward relation.
theorem relation_power_positive_pushforward_reverse_of_injective[A, B](f: A -> B, r: (A, A) -> Bool, n: Nat) {
    is_injective_fn(f) implies
    relation_subset(
        relation_power(relation_pushforward(f, r), n.suc),
        relation_pushforward(f, relation_power(r, n.suc))
    )
} by {
    define p(k: Nat) -> Bool {
        is_injective_fn(f) implies
        relation_subset(
            relation_power(relation_pushforward(f, r), k.suc),
            relation_pushforward(f, relation_power(r, k.suc))
        )
    }
    relation_power_positive_pushforward_reverse_base(f, r)
    p(Nat.zero)
    forall(k: Nat) {
        if p(k) {
            if is_injective_fn(f) {
                p(k)
                relation_subset(
                    relation_power(relation_pushforward(f, r), k.suc),
                    relation_pushforward(f, relation_power(r, k.suc))
                )
                let source = relation_power(relation_pushforward(f, r), k.suc.suc)
                let target = relation_pushforward(f, relation_power(r, k.suc.suc))
                forall(u: B, v: B) {
                    if source(u, v) {
                        relation_power_positive_pushforward_reverse_step_of_injective_at(f, r, k, u, v)
                        target(u, v)
                    }
                }
                relation_subset(source, target)
                relation_subset(
                    relation_power(relation_pushforward(f, r), k.suc.suc),
                    relation_pushforward(f, relation_power(r, k.suc.suc))
                )
                p(k.suc)
            }
        }
    }
    forall(k: Nat) { p(k) implies p(k.suc) }
    p(Nat.zero) and forall(k: Nat) { p(k) implies p(k.suc) }
    alt_induction(p)
    forall(k: Nat) { p(k) }
    p(n)
    if is_injective_fn(f) {
        relation_subset(
            relation_power(relation_pushforward(f, r), n.suc),
            relation_pushforward(f, relation_power(r, n.suc))
        )
    }
}

/// Injective maps give exact transport for each positive relation-power instance under pushforward.
theorem relation_power_positive_pushforward_of_injective_at[A, B](
    f: A -> B,
    r: (A, A) -> Bool,
    n: Nat,
    u: B,
    v: B
) {
    is_injective_fn(f) implies
    relation_pushforward(f, relation_power(r, n.suc), u, v) =
    relation_power(relation_pushforward(f, r), n.suc, u, v)
} by {
    if is_injective_fn(f) {
        if relation_pushforward(f, relation_power(r, n.suc), u, v) {
            relation_power_pushforward_subset(f, r, n.suc)
            relation_subset_step(
                relation_pushforward(f, relation_power(r, n.suc)),
                relation_power(relation_pushforward(f, r), n.suc),
                u,
                v
            )
            relation_power(relation_pushforward(f, r), n.suc, u, v)
        }
        if relation_power(relation_pushforward(f, r), n.suc, u, v) {
            relation_power_positive_pushforward_reverse_of_injective(f, r, n)
            relation_subset_step(
                relation_power(relation_pushforward(f, r), n.suc),
                relation_pushforward(f, relation_power(r, n.suc)),
                u,
                v
            )
            relation_pushforward(f, relation_power(r, n.suc), u, v)
        }
        relation_pushforward(f, relation_power(r, n.suc), u, v) =
        relation_power(relation_pushforward(f, r), n.suc, u, v)
    }
}

/// The pushforward of a reflexive-transitive closure lies in the closure of the pushforward.
theorem relation_refl_trans_closure_pushforward_subset[A, B](f: A -> B, r: (A, A) -> Bool) {
    relation_subset(
        relation_pushforward(f, relation_refl_trans_closure(r)),
        relation_refl_trans_closure(relation_pushforward(f, r))
    )
} by {
    forall(u: B, v: B) {
        if relation_pushforward(f, relation_refl_trans_closure(r), u, v) {
            relation_pushforward_has_preimages(f, relation_refl_trans_closure(r), u, v)
            let x: A satisfy {
                exists(y0: A) {
                    f(x) = u and f(y0) = v and relation_refl_trans_closure(r, x, y0)
                }
            }
            let y: A satisfy {
                f(x) = u and f(y) = v and relation_refl_trans_closure(r, x, y)
            }
            let n: Nat satisfy {
                relation_power(r, n, x, y)
            }
            exists(a: A, b: A) {
                a = x and b = y and f(a) = u and f(b) = v and relation_power(r, n, a, b)
            }
            relation_pushforward(f, relation_power(r, n), u, v)
            relation_power_pushforward_subset(f, r, n)
            relation_subset_step(
                relation_pushforward(f, relation_power(r, n)),
                relation_power(relation_pushforward(f, r), n),
                u,
                v
            )
            relation_power(relation_pushforward(f, r), n, u, v)
            relation_power_in_refl_trans_closure(relation_pushforward(f, r), n, u, v)
            relation_refl_trans_closure(relation_pushforward(f, r), u, v)
        }
    }
}

/// The pushforward of a transitive closure lies in the transitive closure of the pushforward.
theorem relation_trans_closure_pushforward_subset[A, B](f: A -> B, r: (A, A) -> Bool) {
    relation_subset(
        relation_pushforward(f, relation_trans_closure(r)),
        relation_trans_closure(relation_pushforward(f, r))
    )
} by {
    forall(u: B, v: B) {
        if relation_pushforward(f, relation_trans_closure(r), u, v) {
            relation_pushforward_has_preimages(f, relation_trans_closure(r), u, v)
            let x: A satisfy {
                exists(y0: A) {
                    f(x) = u and f(y0) = v and relation_trans_closure(r, x, y0)
                }
            }
            let y: A satisfy {
                f(x) = u and f(y) = v and relation_trans_closure(r, x, y)
            }
            let n: Nat satisfy {
                relation_power(r, n.suc, x, y)
            }
            exists(a: A, b: A) {
                a = x and b = y and f(a) = u and f(b) = v and relation_power(r, n.suc, a, b)
            }
            relation_pushforward(f, relation_power(r, n.suc), u, v)
            relation_power_pushforward_subset(f, r, n.suc)
            relation_subset_step(
                relation_pushforward(f, relation_power(r, n.suc)),
                relation_power(relation_pushforward(f, r), n.suc),
                u,
                v
            )
            relation_power(relation_pushforward(f, r), n.suc, u, v)
            relation_power_positive_in_trans_closure(relation_pushforward(f, r), n, u, v)
            relation_trans_closure(relation_pushforward(f, r), u, v)
        }
    }
}

/// Injective maps give exact transport for each transitive-closure instance under pushforward.
theorem relation_trans_closure_pushforward_of_injective_at[A, B](f: A -> B, r: (A, A) -> Bool, u: B, v: B) {
    is_injective_fn(f) implies
    relation_pushforward(f, relation_trans_closure(r), u, v) = relation_trans_closure(relation_pushforward(f, r), u, v)
} by {
    if is_injective_fn(f) {
        if relation_pushforward(f, relation_trans_closure(r), u, v) {
            relation_trans_closure_pushforward_subset(f, r)
            relation_subset_step(
                relation_pushforward(f, relation_trans_closure(r)),
                relation_trans_closure(relation_pushforward(f, r)),
                u,
                v
            )
            relation_trans_closure(relation_pushforward(f, r), u, v)
        }
        if relation_trans_closure(relation_pushforward(f, r), u, v) {
            let n: Nat satisfy {
                relation_power(relation_pushforward(f, r), n.suc, u, v)
            }
            relation_power_positive_pushforward_reverse_of_injective(f, r, n)
            relation_subset_step(
                relation_power(relation_pushforward(f, r), n.suc),
                relation_pushforward(f, relation_power(r, n.suc)),
                u,
                v
            )
            relation_pushforward(f, relation_power(r, n.suc), u, v)
            relation_pushforward_has_preimages(f, relation_power(r, n.suc), u, v)
            let x: A satisfy {
                exists(y0: A) {
                    f(x) = u and f(y0) = v and relation_power(r, n.suc, x, y0)
                }
            }
            let y: A satisfy {
                f(x) = u and f(y) = v and relation_power(r, n.suc, x, y)
            }
            relation_power_positive_in_trans_closure(r, n, x, y)
            relation_trans_closure(r, x, y)
            exists(a: A, b: A) {
                a = x and b = y and f(a) = u and f(b) = v and relation_trans_closure(r, a, b)
            }
            relation_pushforward(f, relation_trans_closure(r), u, v)
        }
        relation_pushforward(f, relation_trans_closure(r), u, v) = relation_trans_closure(relation_pushforward(f, r), u, v)
    }
}

/// Bijective maps give exact transport for each reflexive-transitive-closure instance under pushforward.
theorem relation_refl_trans_closure_pushforward_of_bijection_at[A, B](f: A -> B, r: (A, A) -> Bool, u: B, v: B) {
    is_bijection_fn(f) implies
    relation_pushforward(f, relation_refl_trans_closure(r), u, v) =
    relation_refl_trans_closure(relation_pushforward(f, r), u, v)
} by {
    if is_bijection_fn(f) {
        bijection_fn_is_injective(f)
        bijection_fn_is_surjective(f)
        if relation_pushforward(f, relation_refl_trans_closure(r), u, v) {
            relation_refl_trans_closure_pushforward_subset(f, r)
            relation_subset_step(
                relation_pushforward(f, relation_refl_trans_closure(r)),
                relation_refl_trans_closure(relation_pushforward(f, r)),
                u,
                v
            )
            relation_refl_trans_closure(relation_pushforward(f, r), u, v)
        }
        if relation_refl_trans_closure(relation_pushforward(f, r), u, v) {
            let n: Nat satisfy {
                relation_power(relation_pushforward(f, r), n, u, v)
            }
            zero_or_suc(n)
            if n = Nat.zero {
                relation_power(relation_pushforward(f, r), Nat.zero, u, v)
                relation_power_zero(relation_pushforward(f, r))
                eq_relation[B](u, v)
                u = v
                surjective_fn_has_preimage(f, u)
                let x: A satisfy {
                    f(x) = u
                }
                relation_power_zero_self(r, x)
                relation_power_in_refl_trans_closure(r, Nat.zero, x, x)
                relation_refl_trans_closure(r, x, x)
                exists(a: A, b: A) {
                    a = x and b = x and f(a) = u and f(b) = v and relation_refl_trans_closure(r, a, b)
                }
                relation_pushforward(f, relation_refl_trans_closure(r), u, v)
            } else {
                let k: Nat satisfy {
                    n = k.suc
                }
                relation_power(relation_pushforward(f, r), k.suc, u, v)
                relation_power_positive_pushforward_reverse_of_injective(f, r, k)
                relation_subset_step(
                    relation_power(relation_pushforward(f, r), k.suc),
                    relation_pushforward(f, relation_power(r, k.suc)),
                    u,
                    v
                )
                relation_pushforward(f, relation_power(r, k.suc), u, v)
                relation_pushforward_has_preimages(f, relation_power(r, k.suc), u, v)
                let x: A satisfy {
                    exists(y0: A) {
                        f(x) = u and f(y0) = v and relation_power(r, k.suc, x, y0)
                    }
                }
                let y: A satisfy {
                    f(x) = u and f(y) = v and relation_power(r, k.suc, x, y)
                }
                relation_power_in_refl_trans_closure(r, k.suc, x, y)
                relation_refl_trans_closure(r, x, y)
                exists(a: A, b: A) {
                    a = x and b = y and f(a) = u and f(b) = v and relation_refl_trans_closure(r, a, b)
                }
                relation_pushforward(f, relation_refl_trans_closure(r), u, v)
            }
        }
        relation_pushforward(f, relation_refl_trans_closure(r), u, v) =
        relation_refl_trans_closure(relation_pushforward(f, r), u, v)
    }
}

/// The image of a relation support is contained in the support of the pushed-forward relation.
theorem relation_pushforward_support_image_subset[A, B](f: A -> B, r: (A, A) -> Bool) {
    set_image(relation_support(r), f).subset(relation_support(relation_pushforward(f, r)))
} by {
    let lhs = set_image(relation_support(r), f)
    let rhs = relation_support(relation_pushforward(f, r))
    forall(u: B) {
        if lhs.contains(u) {
            set_image_contains_witness(relation_support(r), f, u)
            let x: A satisfy {
                relation_support(r).contains(x) and u = f(x)
            }
            relation_support_contains_witness(r, x)
            let y: A satisfy {
                r(x, y)
            }
            relation_pushforward_intro(f, r, x, y)
            relation_pushforward(f, r, f(x), f(y))
            relation_support_contains_of_related_left(relation_pushforward(f, r), f(x), f(y))
            rhs.contains(f(x))
            rhs.contains(u)
        }
    }
    subset_contains_eq(lhs, rhs)
    lhs.subset(rhs)
}

/// The support of a pushed-forward relation is contained in the image of source support.
theorem relation_pushforward_support_image_reverse[A, B](f: A -> B, r: (A, A) -> Bool) {
    relation_support(relation_pushforward(f, r)).subset(set_image(relation_support(r), f))
} by {
    let lhs = set_image(relation_support(r), f)
    let rhs = relation_support(relation_pushforward(f, r))
    forall(u: B) {
        if rhs.contains(u) {
            relation_support_contains_witness(relation_pushforward(f, r), u)
            let v: B satisfy {
                relation_pushforward(f, r, u, v)
            }
            relation_pushforward_has_preimages(f, r, u, v)
            let x: A satisfy {
                exists(y0: A) {
                    f(x) = u and f(y0) = v and r(x, y0)
                }
            }
            let y: A satisfy {
                f(x) = u and f(y) = v and r(x, y)
            }
            relation_support_contains_of_related_left(r, x, y)
            relation_support(r).contains(x)
            maps_into_set_image(relation_support(r), f, x)
            lhs.contains(f(x))
            lhs.contains(u)
        }
    }
    subset_contains_eq(rhs, lhs)
    rhs.subset(lhs)
}

/// The support of a pushed-forward relation is the image of the source support.
theorem relation_pushforward_support_image_eq[A, B](f: A -> B, r: (A, A) -> Bool) {
    set_image(relation_support(r), f) = relation_support(relation_pushforward(f, r))
} by {
    relation_pushforward_support_image_subset(f, r)
    relation_pushforward_support_image_reverse(f, r)
    double_inclusion(set_image(relation_support(r), f), relation_support(relation_pushforward(f, r)))
}

/// Pulling back a relation sends support into the preimage of the target support.
theorem relation_pullback_support_subset_preimage[A, B](f: A -> B, r: (B, B) -> Bool) {
    relation_support(relation_pullback(f, r)).subset(set_preimage(f, relation_support(r)))
} by {
    forall(x: A) {
        if relation_support(relation_pullback(f, r)).contains(x) {
            relation_support_contains_witness(relation_pullback(f, r), x)
            let y: A satisfy {
                relation_pullback(f, r, x, y)
            }
            relation_pullback(f, r, x, y) = r(f(x), f(y))
            relation_support_contains_of_related_left(r, f(x), f(y))
            relation_support(r).contains(f(x))
            set_preimage_contains_eq(f, relation_support(r), x)
            set_preimage(f, relation_support(r)).contains(x)
        }
    }
}

/// Surjective pullback contains the preimage of target support.
theorem relation_pullback_support_preimage_reverse_of_surjective[A, B](f: A -> B, r: (B, B) -> Bool) {
    is_surjective_fn(f) implies
    set_preimage(f, relation_support(r)).subset(relation_support(relation_pullback(f, r)))
} by {
    let lhs = set_preimage(f, relation_support(r))
    let rhs = relation_support(relation_pullback(f, r))
    if is_surjective_fn(f) {
        forall(x: A) {
            if lhs.contains(x) {
                set_preimage_contains_eq(f, relation_support(r), x)
                relation_support(r).contains(f(x))
                relation_support_contains_witness(r, f(x))
                let y: B satisfy {
                    r(f(x), y)
                }
                surjective_fn_has_preimage(f, y)
                let z: A satisfy {
                    f(z) = y
                }
                relation_pullback(f, r, x, z) = r(f(x), f(z))
                relation_pullback(f, r, x, z)
                relation_support_contains_of_related_left(relation_pullback(f, r), x, z)
                rhs.contains(x)
            }
        }
        subset_contains_eq(lhs, rhs)
        lhs.subset(rhs)
    }
}

/// The image of an equivalence class is contained in the pushed-forward class of its representative.
theorem set_image_equivalence_class_pushforward_subset[A, B](f: A -> B, r: (A, A) -> Bool, a: A) {
    set_image(equivalence_class(r, a), f).subset(equivalence_class(relation_pushforward(f, r), f(a)))
} by {
    forall(u: B) {
        if set_image(equivalence_class(r, a), f).contains(u) {
            set_image_contains_witness(equivalence_class(r, a), f, u)
            let x: A satisfy {
                equivalence_class(r, a).contains(x) and u = f(x)
            }
            equivalence_class_contains_eq(r, a, x)
            r(a, x)
            relation_pushforward_intro(f, r, a, x)
            relation_pushforward(f, r, f(a), f(x))
            equivalence_class_contains_eq(relation_pushforward(f, r), f(a), u)
            equivalence_class(relation_pushforward(f, r), f(a)).contains(u)
        }
    }
}

/// A class for a pulled-back relation is the preimage of the corresponding target class.
theorem relation_pullback_equivalence_class_eq_preimage[A, B](f: A -> B, r: (B, B) -> Bool, a: A) {
    equivalence_class(relation_pullback(f, r), a) = set_preimage(f, equivalence_class(r, f(a)))
} by {
    let lhs = equivalence_class(relation_pullback(f, r), a)
    let rhs = set_preimage(f, equivalence_class(r, f(a)))
    forall(x: A) {
        if lhs.contains(x) {
            equivalence_class_contains_eq(relation_pullback(f, r), a, x)
            relation_pullback(f, r, a, x)
            relation_pullback(f, r, a, x) = r(f(a), f(x))
            equivalence_class_contains_eq(r, f(a), f(x))
            equivalence_class(r, f(a)).contains(f(x))
            set_preimage_contains_eq(f, equivalence_class(r, f(a)), x)
            rhs.contains(x)
        }
        if rhs.contains(x) {
            set_preimage_contains_eq(f, equivalence_class(r, f(a)), x)
            equivalence_class(r, f(a)).contains(f(x))
            equivalence_class_contains_eq(r, f(a), f(x))
            r(f(a), f(x))
            relation_pullback(f, r, a, x)
            equivalence_class_contains_eq(relation_pullback(f, r), a, x)
            lhs.contains(x)
        }
        lhs.contains(x) = rhs.contains(x)
    }
    set_ext(lhs, rhs)
}
