from data.basic.set import Set, set_image, maps_into_set_image, set_image_contains_witness
from data.basic.functions import is_injective_fn, is_bijection_fn, bijection_fn_is_injective,
    injective_fn_eq

/// If two images are disjoint, then the source sets are disjoint.
theorem set_image_disjoint_imp_disjoint[T, U](a: Set[T], b: Set[T], f: T -> U) {
    set_image(a, f).is_disjoint(set_image(b, f)) implies a.is_disjoint(b)
} by {
    if set_image(a, f).is_disjoint(set_image(b, f)) {
        forall(x: T) {
            if a.contains(x) and b.contains(x) {
                maps_into_set_image(a, f, x)
                maps_into_set_image(b, f, x)
                set_image(a, f).contains(f(x))
                set_image(b, f).contains(f(x))
                set_image(a, f).is_disjoint(set_image(b, f)) = forall(y: U) {
                    not (set_image(a, f).contains(y) and set_image(b, f).contains(y))
                }
                not (set_image(a, f).contains(f(x)) and set_image(b, f).contains(f(x)))
                false
            }
            not (a.contains(x) and b.contains(x))
        }
        a.is_disjoint(b)
    }
}

/// An injective map sends disjoint source sets to disjoint image sets.
theorem set_image_disjoint_of_injective[T, U](a: Set[T], b: Set[T], f: T -> U) {
    is_injective_fn(f) and a.is_disjoint(b) implies
        set_image(a, f).is_disjoint(set_image(b, f))
} by {
    if is_injective_fn(f) and a.is_disjoint(b) {
        forall(y: U) {
            if set_image(a, f).contains(y) and set_image(b, f).contains(y) {
                set_image_contains_witness(a, f, y)
                let x: T satisfy {
                    a.contains(x) and y = f(x)
                }
                set_image_contains_witness(b, f, y)
                let z: T satisfy {
                    b.contains(z) and y = f(z)
                }
                y = f(x)
                y = f(z)
                f(x) = f(z)
                injective_fn_eq(f, x, z)
                x = z
                b.contains(x)
                a.is_disjoint(b) = forall(t: T) {
                    not (a.contains(t) and b.contains(t))
                }
                not (a.contains(x) and b.contains(x))
                false
            }
            not (set_image(a, f).contains(y) and set_image(b, f).contains(y))
        }
        set_image(a, f).is_disjoint(set_image(b, f))
    }
}

/// Under an injective map, disjointness of images is equivalent to disjointness of sources.
theorem set_image_disjoint_iff_of_injective[T, U](a: Set[T], b: Set[T], f: T -> U) {
    is_injective_fn(f) implies
        (set_image(a, f).is_disjoint(set_image(b, f)) = a.is_disjoint(b))
} by {
    if is_injective_fn(f) {
        if set_image(a, f).is_disjoint(set_image(b, f)) {
            set_image_disjoint_imp_disjoint(a, b, f)
            a.is_disjoint(b)
        }
        if a.is_disjoint(b) {
            set_image_disjoint_of_injective(a, b, f)
            set_image(a, f).is_disjoint(set_image(b, f))
        }
        set_image(a, f).is_disjoint(set_image(b, f)) = a.is_disjoint(b)
    }
}

/// Under a bijection, disjointness of images is equivalent to disjointness of sources.
theorem set_image_disjoint_iff_of_bijection[T, U](a: Set[T], b: Set[T], f: T -> U) {
    is_bijection_fn(f) implies
        (set_image(a, f).is_disjoint(set_image(b, f)) = a.is_disjoint(b))
} by {
    if is_bijection_fn(f) {
        bijection_fn_is_injective(f)
        is_injective_fn(f)
        set_image_disjoint_iff_of_injective(a, b, f)
        set_image(a, f).is_disjoint(set_image(b, f)) = a.is_disjoint(b)
    }
}
