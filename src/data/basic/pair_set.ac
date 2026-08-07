from pair import Pair, pair_map
from data.basic.set import Set

/// True if a pair lies in the product of two sets.
define pair_set_contains[T, U](left: Set[T], right: Set[U], p: Pair[T, U]) -> Bool {
    left.contains(p.first) and right.contains(p.second)
}

/// The Cartesian product of two sets.
define pair_set[T, U](left: Set[T], right: Set[U]) -> Set[Pair[T, U]] {
    Set[Pair[T, U]].new(pair_set_contains(left, right))
}

/// Membership in a Cartesian product is componentwise membership.
theorem pair_set_contains_eq[T, U](left: Set[T], right: Set[U], p: Pair[T, U]) {
    pair_set(left, right).contains(p) =
    (left.contains(p.first) and right.contains(p.second))
}

/// A pair whose components lie in two sets lies in their Cartesian product.
theorem pair_set_contains_of_components[T, U](left: Set[T], right: Set[U], x: T, y: U) {
    left.contains(x) and right.contains(y) implies
    pair_set(left, right).contains(Pair.new(x, y))
} by {
    if left.contains(x) and right.contains(y) {
        let p = Pair.new(x, y)
        p.first = x
        p.second = y
        pair_set_contains_eq(left, right, p)
        pair_set(left, right).contains(p)
    }
}

/// Membership in a Cartesian product gives membership of the first component.
theorem pair_set_contains_imp_left[T, U](left: Set[T], right: Set[U], p: Pair[T, U]) {
    pair_set(left, right).contains(p) implies left.contains(p.first)
} by {
    if pair_set(left, right).contains(p) {
        pair_set_contains_eq(left, right, p)
        left.contains(p.first)
    }
}

/// Membership in a Cartesian product gives membership of the second component.
theorem pair_set_contains_imp_right[T, U](left: Set[T], right: Set[U], p: Pair[T, U]) {
    pair_set(left, right).contains(p) implies right.contains(p.second)
} by {
    if pair_set(left, right).contains(p) {
        pair_set_contains_eq(left, right, p)
        right.contains(p.second)
    }
}

/// Pairing two set-preserving functions preserves Cartesian-product membership.
theorem pair_map_preserves_pair_set[T, U, V, W](left: Set[T], right: Set[U],
    left_target: Set[V], right_target: Set[W], f: T -> V, g: U -> W, p: Pair[T, U]) {
    pair_set(left, right).contains(p) and
    (forall(x: T) { left.contains(x) implies left_target.contains(f(x)) }) and
    (forall(y: U) { right.contains(y) implies right_target.contains(g(y)) })
    implies pair_set(left_target, right_target).contains(pair_map(f, g, p))
} by {
    if pair_set(left, right).contains(p) and
        (forall(x: T) { left.contains(x) implies left_target.contains(f(x)) }) and
        (forall(y: U) { right.contains(y) implies right_target.contains(g(y)) }) {
        pair_set_contains_imp_left(left, right, p)
        pair_set_contains_imp_right(left, right, p)
        left.contains(p.first)
        right.contains(p.second)
        left_target.contains(f(p.first))
        right_target.contains(g(p.second))
        pair_map(f, g, p).first = f(p.first)
        pair_map(f, g, p).second = g(p.second)
        pair_set_contains_eq(left_target, right_target, pair_map(f, g, p))
        pair_set(left_target, right_target).contains(pair_map(f, g, p))
    }
}
