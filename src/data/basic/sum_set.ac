from data.basic.set import Set, maps_into_set_image, set_ext, set_image, set_image_contains_witness,
    subset_contains
from sum_type import Sum, sum_map, sum_swap

/// True if a sum value lies in the union of two typed sets.
define sum_set_contains[T, U](left: Set[T], right: Set[U], s: Sum[T, U]) -> Bool {
    match s {
        Sum.inl(x) {
            left.contains(x)
        }
        Sum.inr(y) {
            right.contains(y)
        }
    }
}

/// The disjoint union of two sets.
define sum_set[T, U](left: Set[T], right: Set[U]) -> Set[Sum[T, U]] {
    Set[Sum[T, U]].new(sum_set_contains(left, right))
}

/// Membership of a left injection in a disjoint union is membership in the left set.
theorem sum_set_contains_inl[T, U](left: Set[T], right: Set[U], x: T) {
    sum_set(left, right).contains(Sum.inl[T, U](x)) = left.contains(x)
}

/// Membership of a right injection in a disjoint union is membership in the right set.
theorem sum_set_contains_inr[T, U](left: Set[T], right: Set[U], y: U) {
    sum_set(left, right).contains(Sum.inr[T, U](y)) = right.contains(y)
}

/// A left value whose component lies in a set lies in the disjoint union.
theorem sum_set_contains_left_of_component[T, U](left: Set[T], right: Set[U], x: T) {
    left.contains(x) implies sum_set(left, right).contains(Sum.inl[T, U](x))
} by {
    if left.contains(x) {
        sum_set_contains_inl(left, right, x)
        sum_set(left, right).contains(Sum.inl[T, U](x))
    }
}

/// A right value whose component lies in a set lies in the disjoint union.
theorem sum_set_contains_right_of_component[T, U](left: Set[T], right: Set[U], y: U) {
    right.contains(y) implies sum_set(left, right).contains(Sum.inr[T, U](y))
} by {
    if right.contains(y) {
        sum_set_contains_inr(left, right, y)
        sum_set(left, right).contains(Sum.inr[T, U](y))
    }
}

/// Membership of a left value in a disjoint union gives membership of its component.
theorem sum_set_contains_inl_imp_left[T, U](left: Set[T], right: Set[U], x: T) {
    sum_set(left, right).contains(Sum.inl[T, U](x)) implies left.contains(x)
} by {
    if sum_set(left, right).contains(Sum.inl[T, U](x)) {
        sum_set_contains_inl(left, right, x)
        left.contains(x)
    }
}

/// Membership of a right value in a disjoint union gives membership of its component.
theorem sum_set_contains_inr_imp_right[T, U](left: Set[T], right: Set[U], y: U) {
    sum_set(left, right).contains(Sum.inr[T, U](y)) implies right.contains(y)
} by {
    if sum_set(left, right).contains(Sum.inr[T, U](y)) {
        sum_set_contains_inr(left, right, y)
        right.contains(y)
    }
}

/// Mapping two set-preserving functions preserves disjoint-union membership.
theorem sum_map_preserves_sum_set[T, U, V, W](left: Set[T], right: Set[U],
    left_target: Set[V], right_target: Set[W], f: T -> V, g: U -> W, s: Sum[T, U]) {
    sum_set(left, right).contains(s) and
    (forall(x: T) { left.contains(x) implies left_target.contains(f(x)) }) and
    (forall(y: U) { right.contains(y) implies right_target.contains(g(y)) })
    implies sum_set(left_target, right_target).contains(sum_map(s, f, g))
} by {
    if sum_set(left, right).contains(s) and
        (forall(x: T) { left.contains(x) implies left_target.contains(f(x)) }) and
        (forall(y: U) { right.contains(y) implies right_target.contains(g(y)) }) {
        match s {
            Sum.inl(x) {
                sum_set_contains_inl_imp_left(left, right, x)
                left.contains(x)
                left_target.contains(f(x))
                sum_map(s, f, g) = Sum.inl[V, W](f(x))
                sum_set_contains_inl(left_target, right_target, f(x))
                sum_set(left_target, right_target).contains(sum_map(s, f, g))
            }
            Sum.inr(y) {
                sum_set_contains_inr_imp_right(left, right, y)
                right.contains(y)
                right_target.contains(g(y))
                sum_map(s, f, g) = Sum.inr[V, W](g(y))
                sum_set_contains_inr(left_target, right_target, g(y))
                sum_set(left_target, right_target).contains(sum_map(s, f, g))
            }
        }
    }
}

/// A coproduct-set subset gives a subset of the left components.
theorem sum_set_subset_left[T, U](left: Set[T], right: Set[U], left2: Set[T], right2: Set[U]) {
    sum_set(left, right).subset(sum_set(left2, right2)) implies left.subset(left2)
} by {
    if sum_set(left, right).subset(sum_set(left2, right2)) {
        forall(x: T) {
            if left.contains(x) {
                sum_set_contains_left_of_component(left, right, x)
                sum_set(left, right).contains(Sum.inl[T, U](x))
                subset_contains(sum_set(left, right), sum_set(left2, right2), Sum.inl[T, U](x))
                sum_set(left2, right2).contains(Sum.inl[T, U](x))
                sum_set_contains_inl_imp_left(left2, right2, x)
                left2.contains(x)
            }
        }
    }
}

/// A coproduct-set subset gives a subset of the right components.
theorem sum_set_subset_right[T, U](left: Set[T], right: Set[U], left2: Set[T], right2: Set[U]) {
    sum_set(left, right).subset(sum_set(left2, right2)) implies right.subset(right2)
} by {
    if sum_set(left, right).subset(sum_set(left2, right2)) {
        forall(y: U) {
            if right.contains(y) {
                sum_set_contains_right_of_component(left, right, y)
                sum_set(left, right).contains(Sum.inr[T, U](y))
                subset_contains(sum_set(left, right), sum_set(left2, right2), Sum.inr[T, U](y))
                sum_set(left2, right2).contains(Sum.inr[T, U](y))
                sum_set_contains_inr_imp_right(left2, right2, y)
                right2.contains(y)
            }
        }
    }
}

/// Component subsets assemble into a coproduct-set subset.
theorem sum_set_subset_of_components[T, U](left: Set[T], right: Set[U], left2: Set[T], right2: Set[U]) {
    left.subset(left2) and right.subset(right2) implies sum_set(left, right).subset(sum_set(left2, right2))
} by {
    if left.subset(left2) and right.subset(right2) {
        forall(s: Sum[T, U]) {
            if sum_set(left, right).contains(s) {
                match s {
                    Sum.inl(x) {
                        sum_set_contains_inl_imp_left(left, right, x)
                        left.contains(x)
                        subset_contains(left, left2, x)
                        left2.contains(x)
                        sum_set_contains_left_of_component(left2, right2, x)
                        sum_set(left2, right2).contains(s)
                    }
                    Sum.inr(y) {
                        sum_set_contains_inr_imp_right(left, right, y)
                        right.contains(y)
                        subset_contains(right, right2, y)
                        right2.contains(y)
                        sum_set_contains_right_of_component(left2, right2, y)
                        sum_set(left2, right2).contains(s)
                    }
                }
            }
        }
    }
}

/// A coproduct-set inclusion is exactly inclusion of both component sets.
theorem sum_set_subset_iff[T, U](left: Set[T], right: Set[U], left2: Set[T], right2: Set[U]) {
    sum_set(left, right).subset(sum_set(left2, right2)) = (left.subset(left2) and right.subset(right2))
} by {
    if sum_set(left, right).subset(sum_set(left2, right2)) {
        sum_set_subset_left(left, right, left2, right2)
        sum_set_subset_right(left, right, left2, right2)
        left.subset(left2) and right.subset(right2)
    }
    if left.subset(left2) and right.subset(right2) {
        sum_set_subset_of_components(left, right, left2, right2)
        sum_set(left, right).subset(sum_set(left2, right2))
    }
}

/// Coproduct sets are equal exactly when both component sets are equal.
theorem sum_set_eq_iff[T, U](left: Set[T], right: Set[U], left2: Set[T], right2: Set[U]) {
    (sum_set(left, right) = sum_set(left2, right2)) = (left = left2 and right = right2)
} by {
    if sum_set(left, right) = sum_set(left2, right2) {
        forall(x: T) {
            sum_set_contains_inl(left, right, x)
            sum_set_contains_inl(left2, right2, x)
            sum_set(left, right).contains(Sum.inl[T, U](x)) =
                sum_set(left2, right2).contains(Sum.inl[T, U](x))
            left.contains(x) = left2.contains(x)
        }
        set_ext(left, left2)
        forall(y: U) {
            sum_set_contains_inr(left, right, y)
            sum_set_contains_inr(left2, right2, y)
            sum_set(left, right).contains(Sum.inr[T, U](y)) =
                sum_set(left2, right2).contains(Sum.inr[T, U](y))
            right.contains(y) = right2.contains(y)
        }
        set_ext(right, right2)
        left = left2 and right = right2
    }
    if left = left2 and right = right2 {
        sum_set(left, right) = sum_set(left2, right2)
    }
}

/// The image of a coproduct set under a coproduct map is the coproduct of the images.
theorem sum_set_image_sum_map[T, U, V, W](left: Set[T], right: Set[U], f: T -> V, g: U -> W) {
    set_image(sum_set(left, right), function(s: Sum[T, U]) { sum_map(s, f, g) }) =
        sum_set(set_image(left, f), set_image(right, g))
} by {
    let source = sum_set(left, right)
    let target = sum_set(set_image(left, f), set_image(right, g))
    forall(z: Sum[V, W]) {
        if set_image(source, function(s: Sum[T, U]) { sum_map(s, f, g) }).contains(z) {
            set_image_contains_witness(source, function(s: Sum[T, U]) { sum_map(s, f, g) }, z)
            let s: Sum[T, U] satisfy {
                source.contains(s) and z = sum_map(s, f, g)
            }
            match s {
                Sum.inl(x) {
                    sum_set_contains_inl_imp_left(left, right, x)
                    left.contains(x)
                    maps_into_set_image(left, f, x)
                    set_image(left, f).contains(f(x))
                    sum_map(s, f, g) = Sum.inl[V, W](f(x))
                    z = Sum.inl[V, W](f(x))
                    sum_set_contains_left_of_component(set_image(left, f), set_image(right, g), f(x))
                    target.contains(z)
                }
                Sum.inr(y) {
                    sum_set_contains_inr_imp_right(left, right, y)
                    right.contains(y)
                    maps_into_set_image(right, g, y)
                    set_image(right, g).contains(g(y))
                    sum_map(s, f, g) = Sum.inr[V, W](g(y))
                    z = Sum.inr[V, W](g(y))
                    sum_set_contains_right_of_component(set_image(left, f), set_image(right, g), g(y))
                    target.contains(z)
                }
            }
        }
        if target.contains(z) {
            match z {
                Sum.inl(v) {
                    sum_set_contains_inl_imp_left(set_image(left, f), set_image(right, g), v)
                    set_image(left, f).contains(v)
                    set_image_contains_witness(left, f, v)
                    let x: T satisfy {
                        left.contains(x) and v = f(x)
                    }
                    sum_set_contains_left_of_component(left, right, x)
                    source.contains(Sum.inl[T, U](x))
                    maps_into_set_image(source, function(s: Sum[T, U]) { sum_map(s, f, g) }, Sum.inl[T, U](x))
                    sum_map(Sum.inl[T, U](x), f, g) = Sum.inl[V, W](f(x))
                    z = sum_map(Sum.inl[T, U](x), f, g)
                    set_image(source, function(s: Sum[T, U]) { sum_map(s, f, g) }).contains(z)
                }
                Sum.inr(w) {
                    sum_set_contains_inr_imp_right(set_image(left, f), set_image(right, g), w)
                    set_image(right, g).contains(w)
                    set_image_contains_witness(right, g, w)
                    let y: U satisfy {
                        right.contains(y) and w = g(y)
                    }
                    sum_set_contains_right_of_component(left, right, y)
                    source.contains(Sum.inr[T, U](y))
                    maps_into_set_image(source, function(s: Sum[T, U]) { sum_map(s, f, g) }, Sum.inr[T, U](y))
                    sum_map(Sum.inr[T, U](y), f, g) = Sum.inr[V, W](g(y))
                    z = sum_map(Sum.inr[T, U](y), f, g)
                    set_image(source, function(s: Sum[T, U]) { sum_map(s, f, g) }).contains(z)
                }
            }
        }
        set_image(source, function(s: Sum[T, U]) { sum_map(s, f, g) }).contains(z) = target.contains(z)
    }
    set_ext(set_image(source, function(s: Sum[T, U]) { sum_map(s, f, g) }), target)
}

/// Swapping a coproduct set gives the coproduct set with components interchanged.
theorem sum_set_image_swap[T, U](left: Set[T], right: Set[U]) {
    set_image(sum_set(left, right), sum_swap[T, U]) = sum_set(right, left)
} by {
    let source = sum_set(left, right)
    let target = sum_set(right, left)
    forall(z: Sum[U, T]) {
        if set_image(source, sum_swap[T, U]).contains(z) {
            set_image_contains_witness(source, sum_swap[T, U], z)
            let s: Sum[T, U] satisfy {
                source.contains(s) and z = sum_swap(s)
            }
            match s {
                Sum.inl(x) {
                    sum_set_contains_inl_imp_left(left, right, x)
                    left.contains(x)
                    sum_swap(s) = Sum.inr[U, T](x)
                    z = Sum.inr[U, T](x)
                    sum_set_contains_right_of_component(right, left, x)
                    target.contains(z)
                }
                Sum.inr(y) {
                    sum_set_contains_inr_imp_right(left, right, y)
                    right.contains(y)
                    sum_swap(s) = Sum.inl[U, T](y)
                    z = Sum.inl[U, T](y)
                    sum_set_contains_left_of_component(right, left, y)
                    target.contains(z)
                }
            }
        }
        if target.contains(z) {
            match z {
                Sum.inl(y) {
                    sum_set_contains_inl_imp_left(right, left, y)
                    right.contains(y)
                    sum_set_contains_right_of_component(left, right, y)
                    source.contains(Sum.inr[T, U](y))
                    maps_into_set_image(source, sum_swap[T, U], Sum.inr[T, U](y))
                    sum_swap(Sum.inr[T, U](y)) = Sum.inl[U, T](y)
                    z = sum_swap(Sum.inr[T, U](y))
                    set_image(source, sum_swap[T, U]).contains(z)
                }
                Sum.inr(x) {
                    sum_set_contains_inr_imp_right(right, left, x)
                    left.contains(x)
                    sum_set_contains_left_of_component(left, right, x)
                    source.contains(Sum.inl[T, U](x))
                    maps_into_set_image(source, sum_swap[T, U], Sum.inl[T, U](x))
                    sum_swap(Sum.inl[T, U](x)) = Sum.inr[U, T](x)
                    z = sum_swap(Sum.inl[T, U](x))
                    set_image(source, sum_swap[T, U]).contains(z)
                }
            }
        }
        set_image(source, sum_swap[T, U]).contains(z) = target.contains(z)
    }
    set_ext(set_image(source, sum_swap[T, U]), target)
}
