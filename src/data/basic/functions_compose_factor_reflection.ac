/// Deep composition factor-reflection, cancellation, and bijection transport API for functions.

from data.basic.functions import Inhabited, compose, identity_fn, compose_identity_left, compose_identity_right,
    compose_assoc, inverse_fn, inverse_fn_apply_of_surjective,
    inverse_fn_apply_image_of_bijection, function_extensionality, is_injective_fn,
    is_surjective_fn, is_bijection_fn, bijection_fn_is_injective,
    bijection_fn_is_surjective, compose_cancel_left_of_injective_fn,
    compose_cancel_right_of_surjective_fn,
    compose_injective_fn_imp_left_factor_injective_of_right_surjective,
    compose_surjective_fn_imp_right_factor_surjective_of_left_injective,
    compose_bijection_fn, compose_bijection_fn_and_left_injective_imp_right_bijection,
    compose_bijection_fn_and_right_surjective_imp_left_bijection,
    Bijection, bijection_map_is_bijection, compose_bijection, inverse_bijection,
    inverse_bijection_inverse
from data.basic.functions_bijection_cancellation import compose_bijection_cancel_left,
    compose_bijection_cancel_right, inverse_bijection_eq_imp_eq

/// Postcomposition by a fixed left factor as a map between function spaces.
define postcompose_fn[T, U, V](left: U -> V, f: T -> U) -> T -> V {
    compose(left, f)
}

/// Precomposition by a fixed right factor as a map between function spaces.
define precompose_fn[T, U, V](right: T -> U, f: U -> V) -> T -> V {
    compose(f, right)
}

/// Postcomposition by an identity function is the identity operation on functions.
theorem postcompose_fn_identity[T, U](f: T -> U) {
    postcompose_fn(identity_fn[U], f) = f
} by {
    compose_identity_left(f)
}

/// Precomposition by an identity function is the identity operation on functions.
theorem precompose_fn_identity[T, U](f: T -> U) {
    precompose_fn(identity_fn[T], f) = f
} by {
    compose_identity_right(f)
}

/// Successive postcomposition is postcomposition by the composite left factor.
theorem postcompose_fn_compose[T, U, V, W](left1: V -> W, left2: U -> V, f: T -> U) {
    postcompose_fn(compose(left1, left2), f) = postcompose_fn(left1, postcompose_fn(left2, f))
} by {
    compose_assoc(left1, left2, f)
}

/// Successive precomposition is precomposition by the composite right factor.
theorem precompose_fn_compose[A, B, C, D](right1: A -> B, right2: B -> C, f: C -> D) {
    precompose_fn(compose(right2, right1), f) = precompose_fn(right1, precompose_fn(right2, f))
} by {
    compose_assoc(f, right2, right1)
}

/// Postcomposition and precomposition commute when they act on opposite sides.
theorem postcompose_precompose_fn_commute[S, T, U, V](left: U -> V, right: S -> T, f: T -> U) {
    postcompose_fn(left, precompose_fn(right, f)) = precompose_fn(right, postcompose_fn(left, f))
} by {
    postcompose_fn(left, precompose_fn(right, f)) = compose(left, compose(f, right))
    precompose_fn(right, postcompose_fn(left, f)) = compose(compose(left, f), right)
    compose_assoc(left, f, right)
}

/// Equality after postcomposition by an injective function is exactly equality before postcomposition.
theorem postcompose_fn_eq_iff_of_injective_fn[T, U, V](left: U -> V, f: T -> U, g: T -> U) {
    is_injective_fn(left) implies (postcompose_fn(left, f) = postcompose_fn(left, g)) = (f = g)
} by {
    if is_injective_fn(left) {
        if postcompose_fn(left, f) = postcompose_fn(left, g) {
            compose_cancel_left_of_injective_fn(left, f, g)
            f = g
        }
        if f = g {
            postcompose_fn(left, f) = postcompose_fn(left, g)
        }
        (postcompose_fn(left, f) = postcompose_fn(left, g)) = (f = g)
    }
}

/// Equality after precomposition by a surjective function is exactly equality before precomposition.
theorem precompose_fn_eq_iff_of_surjective_fn[T, U, V](right: T -> U, f: U -> V, g: U -> V) {
    is_surjective_fn(right) implies (precompose_fn(right, f) = precompose_fn(right, g)) = (f = g)
} by {
    if is_surjective_fn(right) {
        if precompose_fn(right, f) = precompose_fn(right, g) {
            compose_cancel_right_of_surjective_fn(right, f, g)
            f = g
        }
        if f = g {
            precompose_fn(right, f) = precompose_fn(right, g)
        }
        (precompose_fn(right, f) = precompose_fn(right, g)) = (f = g)
    }
}

/// Postcomposition by an injective function is injective on function spaces.
theorem postcompose_fn_is_injective_of_injective_fn[T, U, V](left: U -> V) {
    is_injective_fn(left) implies is_injective_fn(postcompose_fn[T, U, V](left))
} by {
    if is_injective_fn(left) {
        forall(f: T -> U, g: T -> U) {
            if postcompose_fn(left, f) = postcompose_fn(left, g) {
                postcompose_fn_eq_iff_of_injective_fn(left, f, g)
                f = g
            }
        }
        is_injective_fn(postcompose_fn[T, U, V](left))
    }
}

/// Precomposition by a surjective function is injective on function spaces.
theorem precompose_fn_is_injective_of_surjective_fn[T, U, V](right: T -> U) {
    is_surjective_fn(right) implies is_injective_fn(precompose_fn[T, U, V](right))
} by {
    if is_surjective_fn(right) {
        forall(f: U -> V, g: U -> V) {
            if precompose_fn(right, f) = precompose_fn(right, g) {
                precompose_fn_eq_iff_of_surjective_fn(right, f, g)
                f = g
            }
        }
        is_injective_fn(precompose_fn[T, U, V](right))
    }
}

/// Postcomposition by a bijection is surjective on function spaces.
theorem postcompose_fn_is_surjective_of_bijection_fn[T, U: Inhabited, V](left: U -> V) {
    is_bijection_fn(left) implies is_surjective_fn(postcompose_fn[T, U, V](left))
} by {
    if is_bijection_fn(left) {
        bijection_fn_is_surjective(left)
        is_surjective_fn(left)
        forall(target: T -> V) {
            let source: T -> U = compose(inverse_fn(left), target)
            forall(x: T) {
                compose(left, source, x) = left(source(x))
                inverse_fn_apply_of_surjective(left, target(x))
                left(inverse_fn(left, target(x))) = target(x)
                postcompose_fn(left, source, x) = target(x)
            }
            function_extensionality(postcompose_fn(left, source), target)
            exists(preimage: T -> U) {
                postcompose_fn(left, preimage) = target
            }
        }
        is_surjective_fn(postcompose_fn[T, U, V](left))
    }
}

/// Precomposition by a bijection is surjective on function spaces.
theorem precompose_fn_is_surjective_of_bijection_fn[T: Inhabited, U, V](right: T -> U) {
    is_bijection_fn(right) implies is_surjective_fn(precompose_fn[T, U, V](right))
} by {
    if is_bijection_fn(right) {
        forall(target: T -> V) {
            let source: U -> V = compose(target, inverse_fn(right))
            forall(x: T) {
                compose(target, inverse_fn(right), right(x)) = target(inverse_fn(right, right(x)))
                inverse_fn_apply_image_of_bijection(right, x)
                precompose_fn(right, source, x) = target(x)
            }
            function_extensionality(precompose_fn(right, source), target)
            exists(preimage: U -> V) {
                precompose_fn(right, preimage) = target
            }
        }
        is_surjective_fn(precompose_fn[T, U, V](right))
    }
}

/// Postcomposition by a bijection is itself a bijection on function spaces.
theorem postcompose_fn_is_bijection_of_bijection_fn[T, U: Inhabited, V](left: U -> V) {
    is_bijection_fn(left) implies is_bijection_fn(postcompose_fn[T, U, V](left))
} by {
    if is_bijection_fn(left) {
        bijection_fn_is_injective(left)
        postcompose_fn_is_injective_of_injective_fn[T, U, V](left)
        postcompose_fn_is_surjective_of_bijection_fn[T, U, V](left)
        is_surjective_fn(postcompose_fn[T, U, V](left))
        is_bijection_fn(postcompose_fn[T, U, V](left))
    }
}

/// Precomposition by a bijection is itself a bijection on function spaces.
theorem precompose_fn_is_bijection_of_bijection_fn[T: Inhabited, U, V](right: T -> U) {
    is_bijection_fn(right) implies is_bijection_fn(precompose_fn[T, U, V](right))
} by {
    if is_bijection_fn(right) {
        bijection_fn_is_surjective(right)
        precompose_fn_is_injective_of_surjective_fn[T, U, V](right)
        precompose_fn_is_surjective_of_bijection_fn[T, U, V](right)
        is_surjective_fn(precompose_fn[T, U, V](right))
        is_bijection_fn(precompose_fn[T, U, V](right))
    }
}

/// Postcomposition by a bundled bijection is a bijection on function spaces.
theorem postcompose_fn_is_bijection_of_bijection[T, U: Inhabited, V](e: Bijection[U, V]) {
    is_bijection_fn(postcompose_fn[T, U, V](e.map))
} by {
    bijection_map_is_bijection(e)
    postcompose_fn_is_bijection_of_bijection_fn[T, U, V](e.map)
}

/// Precomposition by a bundled bijection is a bijection on function spaces.
theorem precompose_fn_is_bijection_of_bijection[T: Inhabited, U, V](e: Bijection[T, U]) {
    is_bijection_fn(precompose_fn[T, U, V](e.map))
} by {
    bijection_map_is_bijection(e)
    precompose_fn_is_bijection_of_bijection_fn[T, U, V](e.map)
}

/// With a bijective right factor, composite bijectivity is equivalent to bijectivity of the left factor.
theorem compose_bijection_fn_iff_left_factor_bijection_of_right_bijection[T, U, V](f: U -> V, g: T -> U) {
    is_bijection_fn(g) implies (is_bijection_fn(compose(f, g)) = is_bijection_fn(f))
} by {
    if is_bijection_fn(g) {
        if is_bijection_fn(compose(f, g)) {
            bijection_fn_is_surjective(g)
            compose_bijection_fn_and_right_surjective_imp_left_bijection(f, g)
            is_bijection_fn(f)
        }
        if is_bijection_fn(f) {
            compose_bijection_fn(f, g)
            is_bijection_fn(compose(f, g))
        }
        is_bijection_fn(compose(f, g)) = is_bijection_fn(f)
    }
}

/// With a bijective left factor, composite bijectivity is equivalent to bijectivity of the right factor.
theorem compose_bijection_fn_iff_right_factor_bijection_of_left_bijection[T, U, V](f: U -> V, g: T -> U) {
    is_bijection_fn(f) implies (is_bijection_fn(compose(f, g)) = is_bijection_fn(g))
} by {
    if is_bijection_fn(f) {
        if is_bijection_fn(compose(f, g)) {
            bijection_fn_is_injective(f)
            compose_bijection_fn_and_left_injective_imp_right_bijection(f, g)
            is_bijection_fn(g)
        }
        if is_bijection_fn(g) {
            compose_bijection_fn(f, g)
            is_bijection_fn(compose(f, g))
        }
        is_bijection_fn(compose(f, g)) = is_bijection_fn(g)
    }
}

/// A fixed left bundled bijection gives an injective map on bundled right factors.
theorem compose_bijection_left_map_is_injective[T, U, V](e: Bijection[U, V]) {
    is_injective_fn(compose_bijection[T, U, V](e))
} by {
    forall(h: Bijection[T, U], k: Bijection[T, U]) {
        if compose_bijection(e, h) = compose_bijection(e, k) {
            compose_bijection_cancel_left(e, h, k)
            h = k
        }
    }
}

/// A fixed right bundled bijection gives an injective map on bundled left factors.
theorem compose_bijection_right_map_is_injective[T, U, V](e: Bijection[T, U]) {
    is_injective_fn(function(h: Bijection[U, V]) { compose_bijection(h, e) })
} by {
    forall(h: Bijection[U, V], k: Bijection[U, V]) {
        if compose_bijection(h, e) = compose_bijection(k, e) {
            compose_bijection_cancel_right(e, h, k)
            h = k
        }
    }
}

/// Taking inverse bijections is injective on bundled bijections.
theorem inverse_bijection_is_injective[T: Inhabited, U: Inhabited] {
    is_injective_fn(inverse_bijection[T, U])
} by {
    forall(e: Bijection[T, U], h: Bijection[T, U]) {
        if inverse_bijection(e) = inverse_bijection(h) {
            inverse_bijection_eq_imp_eq(e, h)
            e = h
        }
    }
}

/// Taking inverse bijections is surjective on bundled bijections.
theorem inverse_bijection_is_surjective[T: Inhabited, U: Inhabited] {
    is_surjective_fn(inverse_bijection[T, U])
} by {
    forall(h: Bijection[U, T]) {
        let e = inverse_bijection(h)
        inverse_bijection_inverse(h)
        exists(preimage: Bijection[T, U]) {
            inverse_bijection(preimage) = h
        }
    }
}

/// Taking inverse bijections is a bijection on bundled bijections.
theorem inverse_bijection_is_bijection[T: Inhabited, U: Inhabited] {
    is_bijection_fn(inverse_bijection[T, U])
} by {
    inverse_bijection_is_injective[T, U]
    inverse_bijection_is_surjective[T, U]
}

/// Injectivity of a composite plus surjectivity of the right factor yields an injective postcomposition operator for the left factor.
theorem postcompose_fn_is_injective_of_compose_injective_and_right_surjective[A, B, C, X](
    f: B -> C, g: A -> B
) {
    is_injective_fn(compose(f, g)) and is_surjective_fn(g)
    implies is_injective_fn(postcompose_fn[X, B, C](f))
} by {
    if is_injective_fn(compose(f, g)) and is_surjective_fn(g) {
        compose_injective_fn_imp_left_factor_injective_of_right_surjective(f, g)
        postcompose_fn_is_injective_of_injective_fn[X, B, C](f)
        is_injective_fn(postcompose_fn[X, B, C](f))
    }
}

/// Surjectivity of a composite plus injectivity of the left factor yields an injective precomposition operator for the right factor.
theorem precompose_fn_is_injective_of_compose_surjective_and_left_injective[A, B, C, X](
    f: B -> C, g: A -> B
) {
    is_surjective_fn(compose(f, g)) and is_injective_fn(f)
    implies is_injective_fn(precompose_fn[A, B, X](g))
} by {
    if is_surjective_fn(compose(f, g)) and is_injective_fn(f) {
        compose_surjective_fn_imp_right_factor_surjective_of_left_injective(f, g)
        precompose_fn_is_injective_of_surjective_fn[A, B, X](g)
        is_injective_fn(precompose_fn[A, B, X](g))
    }
}
