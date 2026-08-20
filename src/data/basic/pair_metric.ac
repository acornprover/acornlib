from real import Real
numerals Real
from real import add_lte_add, lte_trans, lt_zero_imp_neg
from order import le_max_left, le_max_right, max_le_of_le_left_of_le_right, max_lt_iff, max_lte_iff, max_symm
from analysis import MetricSpace, distance_non_negative, distance_self
from pair import Pair, pair_ext, pair_map, pair_map_first_eq, pair_map_second_eq, swap_first, swap_second

/// The sup metric on a pair: the larger of the two componentwise distances.
define pair_distance[M: MetricSpace, N: MetricSpace](p: Pair[M, N], q: Pair[M, N]) -> Real {
    (p.first.distance(q.first)).max(p.second.distance(q.second))
}

/// The first-component distance is at most the pair distance.
theorem first_distance_le_pair_distance[M: MetricSpace, N: MetricSpace](p: Pair[M, N], q: Pair[M, N]) {
    p.first.distance(q.first) <= pair_distance(p, q)
} by {
    le_max_left(p.first.distance(q.first), p.second.distance(q.second))
}

/// The second-component distance is at most the pair distance.
theorem second_distance_le_pair_distance[M: MetricSpace, N: MetricSpace](p: Pair[M, N], q: Pair[M, N]) {
    p.second.distance(q.second) <= pair_distance(p, q)
} by {
    le_max_right(p.first.distance(q.first), p.second.distance(q.second))
}

/// The pair distance is strictly below a bound exactly when both component distances are strictly below it.
theorem pair_distance_lt_iff_components_lt[M: MetricSpace, N: MetricSpace](p: Pair[M, N], q: Pair[M, N], r: Real) {
    pair_distance(p, q) < r = (p.first.distance(q.first) < r and p.second.distance(q.second) < r)
} by {
    max_lt_iff(p.first.distance(q.first), p.second.distance(q.second), r)
}

/// The pair distance is at most a bound exactly when both component distances are at most it.
theorem pair_distance_le_iff_components_le[M: MetricSpace, N: MetricSpace](p: Pair[M, N], q: Pair[M, N], r: Real) {
    pair_distance(p, q) <= r = (p.first.distance(q.first) <= r and p.second.distance(q.second) <= r)
} by {
    max_lte_iff(p.first.distance(q.first), p.second.distance(q.second), r)
}

/// A strict pair-distance bound gives the corresponding first-component bound.
theorem pair_distance_lt_imp_first_distance_lt[M: MetricSpace, N: MetricSpace](p: Pair[M, N], q: Pair[M, N], r: Real) {
    pair_distance(p, q) < r implies p.first.distance(q.first) < r
} by {
    if pair_distance(p, q) < r {
        pair_distance_lt_iff_components_lt(p, q, r)
        p.first.distance(q.first) < r
    }
}

/// A strict pair-distance bound gives the corresponding second-component bound.
theorem pair_distance_lt_imp_second_distance_lt[M: MetricSpace, N: MetricSpace](p: Pair[M, N], q: Pair[M, N], r: Real) {
    pair_distance(p, q) < r implies p.second.distance(q.second) < r
} by {
    if pair_distance(p, q) < r {
        pair_distance_lt_iff_components_lt(p, q, r)
        p.second.distance(q.second) < r
    }
}

/// Strict component bounds give the corresponding pair-distance bound.
theorem pair_distance_lt_of_components_lt[M: MetricSpace, N: MetricSpace](p: Pair[M, N], q: Pair[M, N], r: Real) {
    p.first.distance(q.first) < r and p.second.distance(q.second) < r implies pair_distance(p, q) < r
} by {
    if p.first.distance(q.first) < r and p.second.distance(q.second) < r {
        pair_distance_lt_iff_components_lt(p, q, r)
        pair_distance(p, q) < r
    }
}

/// A pair-distance bound gives the corresponding first-component bound.
theorem pair_distance_le_imp_first_distance_le[M: MetricSpace, N: MetricSpace](p: Pair[M, N], q: Pair[M, N], r: Real) {
    pair_distance(p, q) <= r implies p.first.distance(q.first) <= r
} by {
    if pair_distance(p, q) <= r {
        pair_distance_le_iff_components_le(p, q, r)
        p.first.distance(q.first) <= r
    }
}

/// A pair-distance bound gives the corresponding second-component bound.
theorem pair_distance_le_imp_second_distance_le[M: MetricSpace, N: MetricSpace](p: Pair[M, N], q: Pair[M, N], r: Real) {
    pair_distance(p, q) <= r implies p.second.distance(q.second) <= r
} by {
    if pair_distance(p, q) <= r {
        pair_distance_le_iff_components_le(p, q, r)
        p.second.distance(q.second) <= r
    }
}

/// Component bounds give the corresponding pair-distance bound.
theorem pair_distance_le_of_components_le[M: MetricSpace, N: MetricSpace](p: Pair[M, N], q: Pair[M, N], r: Real) {
    p.first.distance(q.first) <= r and p.second.distance(q.second) <= r implies pair_distance(p, q) <= r
} by {
    if p.first.distance(q.first) <= r and p.second.distance(q.second) <= r {
        pair_distance_le_iff_components_le(p, q, r)
        pair_distance(p, q) <= r
    }
}

/// The pair distance from a point to itself is zero.
theorem pair_distance_self[M: MetricSpace, N: MetricSpace](p: Pair[M, N]) {
    pair_distance(p, p) = 0
} by {
    distance_self(p.first)
    distance_self(p.second)
    p.first.distance(p.first) = 0
    p.second.distance(p.second) = 0
    pair_distance(p, p) = Real.0.max(Real.0)
    Real.0.max(Real.0) = Real.0
}

/// The pair distance is symmetric.
theorem pair_distance_symmetric[M: MetricSpace, N: MetricSpace](p: Pair[M, N], q: Pair[M, N]) {
    pair_distance(p, q) = pair_distance(q, p)
} by {
    p.first.distance(q.first) = q.first.distance(p.first)
    p.second.distance(q.second) = q.second.distance(p.second)
}

/// Swapping both pairs preserves the pair distance.
theorem pair_distance_swap[M: MetricSpace, N: MetricSpace](p: Pair[M, N], q: Pair[M, N]) {
    pair_distance(p.swap, q.swap) = pair_distance(p, q)
} by {
    swap_first(p)
    swap_first(q)
    p.swap.first.distance(q.swap.first) = p.second.distance(q.second)
    swap_second(p)
    swap_second(q)
    p.swap.second.distance(q.swap.second) = p.first.distance(q.first)
    pair_distance(p.swap, q.swap) = p.second.distance(q.second).max(p.first.distance(q.first))
    max_symm(p.first.distance(q.first), p.second.distance(q.second))
    p.second.distance(q.second).max(p.first.distance(q.first)) =
        p.first.distance(q.first).max(p.second.distance(q.second))
    pair_distance(p.swap, q.swap) = pair_distance(p, q)
}

/// True if the two component maps do not increase distances.
define pair_map_components_nonexpansive[M: MetricSpace, N: MetricSpace, P: MetricSpace, Q: MetricSpace](
    f: M -> P,
    g: N -> Q
) -> Bool {
    (forall(x: M, y: M) {
        f(x).distance(f(y)) <= x.distance(y)
    }) and (forall(x: N, y: N) {
        g(x).distance(g(y)) <= x.distance(y)
    })
}

/// Component bounds for mapped pairs give a pair-distance bound.
theorem pair_distance_pair_map_le_of_component_bounds[M: MetricSpace, N: MetricSpace, P: MetricSpace, Q: MetricSpace](
    f: M -> P,
    g: N -> Q,
    p: Pair[M, N],
    q: Pair[M, N],
    r: Real
) {
    f(p.first).distance(f(q.first)) <= r and g(p.second).distance(g(q.second)) <= r implies
        pair_distance(pair_map(f, g, p), pair_map(f, g, q)) <= r
} by {
    if f(p.first).distance(f(q.first)) <= r and g(p.second).distance(g(q.second)) <= r {
        let fp = pair_map(f, g, p)
        let fq = pair_map(f, g, q)
        pair_map_first_eq(f, g, p)
        pair_map_first_eq(f, g, q)
        fp.first.distance(fq.first) = f(p.first).distance(f(q.first))
        fp.first.distance(fq.first) <= r
        pair_map_second_eq(f, g, p)
        pair_map_second_eq(f, g, q)
        fp.second.distance(fq.second) = g(p.second).distance(g(q.second))
        fp.second.distance(fq.second) <= r
        pair_distance_le_of_components_le(fp, fq, r)
        pair_distance(fp, fq) <= r
        pair_distance(pair_map(f, g, p), pair_map(f, g, q)) = pair_distance(fp, fq)
        pair_distance(pair_map(f, g, p), pair_map(f, g, q)) <= r
    }
}

/// Componentwise nonexpansive maps induce a nonexpansive bound for the pair distance.
theorem pair_distance_pair_map_le_of_components_nonexpansive[M: MetricSpace, N: MetricSpace, P: MetricSpace, Q: MetricSpace](
    f: M -> P,
    g: N -> Q,
    p: Pair[M, N],
    q: Pair[M, N]
) {
    pair_map_components_nonexpansive(f, g) implies
        pair_distance(pair_map(f, g, p), pair_map(f, g, q)) <= pair_distance(p, q)
} by {
    if pair_map_components_nonexpansive(f, g) {
        pair_map_components_nonexpansive(f, g) =
            ((forall(x: M, y: M) {
                f(x).distance(f(y)) <= x.distance(y)
            }) and (forall(x: N, y: N) {
                g(x).distance(g(y)) <= x.distance(y)
            }))
        (forall(x: M, y: M) {
            f(x).distance(f(y)) <= x.distance(y)
        })
        f(p.first).distance(f(q.first)) <= p.first.distance(q.first)
        first_distance_le_pair_distance(p, q)
        lte_trans(f(p.first).distance(f(q.first)), p.first.distance(q.first), pair_distance(p, q))
        f(p.first).distance(f(q.first)) <= pair_distance(p, q)
        (forall(x: N, y: N) {
            g(x).distance(g(y)) <= x.distance(y)
        })
        g(p.second).distance(g(q.second)) <= p.second.distance(q.second)
        second_distance_le_pair_distance(p, q)
        lte_trans(g(p.second).distance(g(q.second)), p.second.distance(q.second), pair_distance(p, q))
        g(p.second).distance(g(q.second)) <= pair_distance(p, q)
        pair_distance_pair_map_le_of_component_bounds(f, g, p, q, pair_distance(p, q))
        pair_distance(pair_map(f, g, p), pair_map(f, g, q)) <= pair_distance(p, q)
    }
}

/// A non-negative real that is at most zero equals zero.
theorem nonneg_le_zero_eq_zero(a: Real) {
    not a.is_negative and a <= 0 implies a = 0
} by {
    if not a.is_negative and a <= 0 {
        if a < 0 {
            lt_zero_imp_neg(a)
            a.is_negative
            false
        }
        not a < 0
        0 <= a
        a = 0
    }
}

/// If the pair distance is zero, the two pairs are equal.
theorem pair_distance_zero_imp_eq[M: MetricSpace, N: MetricSpace](p: Pair[M, N], q: Pair[M, N]) {
    pair_distance(p, q) = 0 implies p = q
} by {
    if pair_distance(p, q) = 0 {
        let p1: M = p.first
        let q1: M = q.first
        let p2: N = p.second
        let q2: N = q.second
        first_distance_le_pair_distance(p, q)
        second_distance_le_pair_distance(p, q)
        p1.distance(q1) <= pair_distance(p, q)
        p2.distance(q2) <= pair_distance(p, q)
        pair_distance(p, q) <= 0
        lte_trans(p1.distance(q1), pair_distance(p, q), 0)
        lte_trans(p2.distance(q2), pair_distance(p, q), 0)
        p1.distance(q1) <= 0
        p2.distance(q2) <= 0
        distance_non_negative(p1, q1)
        distance_non_negative(p2, q2)
        nonneg_le_zero_eq_zero(p1.distance(q1))
        nonneg_le_zero_eq_zero(p2.distance(q2))
        p1.distance(q1) = 0
        p2.distance(q2) = 0
        p1 = q1
        p2 = q2
        p.first = q.first
        p.second = q.second
        pair_ext(p, q)
        p = q
    }
}

/// The triangle inequality holds for the pair distance.
theorem pair_distance_triangle[M: MetricSpace, N: MetricSpace](p: Pair[M, N], q: Pair[M, N], r: Pair[M, N]) {
    pair_distance(p, r) <= pair_distance(p, q) + pair_distance(q, r)
} by {
    let sum: Real = pair_distance(p, q) + pair_distance(q, r)
    let p1: M = p.first
    let q1: M = q.first
    let r1: M = r.first
    let p2: N = p.second
    let q2: N = q.second
    let r2: N = r.second
    p1.distance(r1) <= p1.distance(q1) + q1.distance(r1)
    first_distance_le_pair_distance(p, q)
    first_distance_le_pair_distance(q, r)
    p1.distance(q1) <= pair_distance(p, q)
    q1.distance(r1) <= pair_distance(q, r)
    add_lte_add(p1.distance(q1), pair_distance(p, q), q1.distance(r1), pair_distance(q, r))
    p1.distance(q1) + q1.distance(r1) <= sum
    lte_trans(p1.distance(r1), p1.distance(q1) + q1.distance(r1), sum)
    p1.distance(r1) <= sum

    p2.distance(r2) <= p2.distance(q2) + q2.distance(r2)
    second_distance_le_pair_distance(p, q)
    second_distance_le_pair_distance(q, r)
    p2.distance(q2) <= pair_distance(p, q)
    q2.distance(r2) <= pair_distance(q, r)
    add_lte_add(p2.distance(q2), pair_distance(p, q), q2.distance(r2), pair_distance(q, r))
    p2.distance(q2) + q2.distance(r2) <= sum
    lte_trans(p2.distance(r2), p2.distance(q2) + q2.distance(r2), sum)
    p2.distance(r2) <= sum

    max_le_of_le_left_of_le_right(p1.distance(r1), p2.distance(r2), sum)
}

// TODO: register `Pair[M, N]: MetricSpace` once the instance axiom-search overhead
// for `pair_distance` is reduced; the four axioms hold by `pair_distance_self`,
// `pair_distance_zero_imp_eq`, `pair_distance_symmetric`, and `pair_distance_triangle`.
