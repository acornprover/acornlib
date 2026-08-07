from nat import Nat, lt_and_lte, lte_and_lt, lt_or_lte
from list import List, map, map_contains, range_contains_of_lt, range_pigeonhole,
    unique_is_smallest_containing_list
from data.basic.set import Set, finite_constraint
from data.basic.functions import is_injective_fn

numerals Nat

/// True if every value of the map lands in the set.
///
/// Named so that the infinitude criterion below is a two-part statement, and so that the
/// witness supplied by a parametric family can be checked one condition at a time.
define maps_into[T](g: Nat -> T, s: Set[T]) -> Bool {
    forall(i: Nat) {
        s.contains(g(i))
    }
}

/// Each value of such a map lies in the set.
theorem maps_into_apply[T](g: Nat -> T, s: Set[T], i: Nat) {
    maps_into(g, s) implies s.contains(g(i))
} by {
    if maps_into(g, s) {
        maps_into(g, s) = forall(k: Nat) {
            s.contains(g(k))
        }
        forall(k: Nat) {
            s.contains(g(k))
        }
        s.contains(g(i))
    }
}

/// A pointwise membership check gives the condition.
theorem maps_into_intro[T](g: Nat -> T, s: Set[T]) {
    (forall(i: Nat) { s.contains(g(i)) }) implies maps_into(g, s)
} by {
    if forall(i: Nat) { s.contains(g(i)) } {
        maps_into(g, s) = forall(k: Nat) {
            s.contains(g(k))
        }
        maps_into(g, s)
    }
}

/// A set admitting an injection from the naturals is infinite.
///
/// Any covering list would have to hold one more distinct value than its own length, since
/// the injection supplies that many. `src/set.ac` has finiteness closed under union,
/// intersection, difference, and subsets, but nothing relating injections to it, and this is
/// the missing criterion: it is how a family indexed by the naturals is shown to be large.
theorem set_infinite_of_injection[T](g: Nat -> T, s: Set[T]) {
    is_injective_fn(g) and maps_into(g, s) implies s.is_infinite
} by {
    if is_injective_fn(g) and maps_into(g, s) {
        if s.is_finite {
            s.is_finite = finite_constraint(s.contains)
            finite_constraint(s.contains)
            finite_constraint(s.contains) = exists(superset: List[T]) {
                forall(x: T) {
                    s.contains(x) implies superset.contains(x)
                }
            }
            exists(superset: List[T]) {
                forall(x: T) {
                    s.contains(x) implies superset.contains(x)
                }
            }
            let (cover: List[T]) satisfy {
                forall(x: T) {
                    s.contains(x) implies cover.contains(x)
                }
            }
            forall(y: T) {
                if map(cover.length.suc.range, g).contains(y) {
                    map_contains(cover.length.suc.range, g, y)
                    exists(i: Nat) {
                        cover.length.suc.range.contains(i) and y = g(i)
                    }
                    let (i: Nat) satisfy {
                        cover.length.suc.range.contains(i) and y = g(i)
                    }
                    maps_into_apply(g, s, i)
                    s.contains(g(i))
                    s.contains(y)
                    cover.contains(y)
                }
                (map(cover.length.suc.range, g).contains(y) implies cover.contains(y))
            }
            unique_is_smallest_containing_list(map(cover.length.suc.range, g), cover)
            map(cover.length.suc.range, g).unique.length <= cover.length
            cover.length < cover.length.suc
            lte_and_lt(map(cover.length.suc.range, g).unique.length, cover.length,
                cover.length.suc)
            (map(cover.length.suc.range, g).unique.length < cover.length.suc)
            range_pigeonhole(cover.length.suc, g)
            exists(i: Nat, j: Nat) {
                i < j and j < cover.length.suc and g(i) = g(j)
            }
            let (i: Nat, j: Nat) satisfy {
                i < j and j < cover.length.suc and g(i) = g(j)
            }
            is_injective_fn(g) = forall(a: Nat, b: Nat) { g(a) = g(b) implies a = b }
            forall(a: Nat, b: Nat) { g(a) = g(b) implies a = b }
            (g(i) = g(j) implies i = j)
            i = j
            i < i
            false
        }
        not s.is_finite
        s.is_infinite = not s.is_finite
        s.is_infinite
    }
}
