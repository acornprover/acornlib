/// Transport and compatibility lemmas for unbundled relations along functions.

from data.basic.functions import Inhabited, compose, identity_fn, inverse_fn, inverse_fn_apply_of_surjective,
    inverse_fn_apply_image_of_bijection, predicate_extensionality, binary_function_extensionality,
    function_extensionality, is_injective_fn, is_surjective_fn, is_bijection_fn, bijection_fn_is_injective,
    bijection_fn_is_surjective, injective_fn_eq, surjective_fn_has_preimage, Bijection,
    bijection_map_is_bijection, bijection_map_is_injective, bijection_map_is_surjective
from algebra.add import Add
from algebra.mul import Mul
from lte import LTE
from data.basic.relation_basic import eq_relation, is_reflexive, is_irreflexive, is_symmetric, is_asymmetric, is_transitive, is_total, is_antisymmetric, is_equivalence, is_partial_equivalence, relation_intersection, relation_union, relation_subset, relation_subset_refl, relation_subset_step, relation_subset_trans, relation_converse, relation_reflexive_closure, relation_symmetric_closure, eq_relation_is_equivalence, relation_intersection_is_equivalence, relation_converse_is_equivalence, equivalence_is_reflexive, equivalence_is_symmetric, equivalence_is_transitive, partial_equivalence_is_symmetric, partial_equivalence_is_transitive, reflexive_self, symmetric_flip, transitive_step, antisymmetric_eq

/// The pullback of a homogeneous relation along a function.
define relation_pullback[A, B](f: A -> B, r: (B, B) -> Bool, x: A, y: A) -> Bool {
    r(f(x), f(y))
}

/// The image of a homogeneous relation along a function.
define relation_pushforward[A, B](f: A -> B, r: (A, A) -> Bool, u: B, v: B) -> Bool {
    exists(x: A, y: A) {
        f(x) = u and f(y) = v and r(x, y)
    }
}

/// True if every element satisfying `p` also satisfies `q`.
define predicate_subset[T](p: T -> Bool, q: T -> Bool) -> Bool {
    forall(x: T) {
        p(x) implies q(x)
    }
}

/// The pullback of a predicate along a function.
define predicate_pullback[A, B](f: A -> B, p: B -> Bool, x: A) -> Bool {
    p(f(x))
}

/// The image of a predicate along a function.
define predicate_pushforward[A, B](f: A -> B, p: A -> Bool, y: B) -> Bool {
    exists(x: A) {
        f(x) = y and p(x)
    }
}

/// A predicate inclusion applies to an explicitly satisfying element.
theorem predicate_subset_step[T](p: T -> Bool, q: T -> Bool, x: T) {
    predicate_subset(p, q) and p(x) implies q(x)
} by {
    if predicate_subset(p, q) and p(x) {
        predicate_subset(p, q) = forall(y: T) {
            p(y) implies q(y)
        }
        q(x)
    }
}

/// Every predicate is contained in itself.
theorem predicate_subset_refl[T](p: T -> Bool) {
    predicate_subset(p, p)
} by {
    forall(x: T) {
        if p(x) {
            p(x)
        }
    }
}

/// Predicate inclusion is transitive.
theorem predicate_subset_trans[T](p: T -> Bool, q: T -> Bool, r: T -> Bool) {
    predicate_subset(p, q) and predicate_subset(q, r) implies predicate_subset(p, r)
} by {
    if predicate_subset(p, q) and predicate_subset(q, r) {
        forall(x: T) {
            if p(x) {
                predicate_subset_step(p, q, x)
                q(x)
                predicate_subset_step(q, r, x)
                r(x)
            }
        }
    }
}

/// Predicate equality is equivalent to mutual inclusion.
theorem predicate_eq_iff_subset_both[T](p: T -> Bool, q: T -> Bool) {
    p = q = (predicate_subset(p, q) and predicate_subset(q, p))
} by {
    if p = q {
        predicate_subset_refl(p)
        predicate_subset(p, q)
        predicate_subset(q, p)
        predicate_subset(p, q) and predicate_subset(q, p)
    }
    if predicate_subset(p, q) and predicate_subset(q, p) {
        forall(x: T) {
            if p(x) {
                predicate_subset_step(p, q, x)
                q(x)
            }
            if q(x) {
                predicate_subset_step(q, p, x)
                p(x)
            }
            p(x) = q(x)
        }
        predicate_extensionality(p, q)
        p = q
    }
    p = q = (predicate_subset(p, q) and predicate_subset(q, p))
}

/// An element satisfying a predicate gives an element satisfying its pushforward.
theorem predicate_pushforward_intro[A, B](f: A -> B, p: A -> Bool, x: A) {
    p(x) implies predicate_pushforward(f, p, f(x))
} by {
    if p(x) {
        exists(a: A) {
            a = x and f(a) = f(x) and p(a)
        }
        predicate_pushforward(f, p, f(x))
    }
}

/// A pushed-forward predicate instance has a preimage satisfying the source predicate.
theorem predicate_pushforward_has_preimage[A, B](f: A -> B, p: A -> Bool, y: B) {
    predicate_pushforward(f, p, y) implies exists(x: A) {
        f(x) = y and p(x)
    }
} by {
    if predicate_pushforward(f, p, y) {
        exists(x: A) {
            f(x) = y and p(x)
        }
    }
}

/// Pulling back along the identity function leaves a predicate unchanged.
theorem predicate_pullback_identity[T](p: T -> Bool) {
    predicate_pullback(identity_fn[T], p) = p
} by {
    let u = predicate_pullback(identity_fn[T], p)
    let v = p
    forall(x: T) {
        u(x) = p(identity_fn[T](x))
        identity_fn[T](x) = x
        u(x) = v(x)
    }
    predicate_extensionality(u, v)
}

/// Pushing forward along the identity function leaves a predicate unchanged.
theorem predicate_pushforward_identity[T](p: T -> Bool) {
    predicate_pushforward(identity_fn[T], p) = p
} by {
    let u = predicate_pushforward(identity_fn[T], p)
    let v = p
    forall(x: T) {
        if u(x) {
            predicate_pushforward_has_preimage(identity_fn[T], p, x)
            let a: T satisfy {
                identity_fn[T](a) = x and p(a)
            }
            identity_fn[T](a) = a
            a = x
            v(x)
        }
        if v(x) {
            predicate_pushforward_intro(identity_fn[T], p, x)
            identity_fn[T](x) = x
            u(x)
        }
        u(x) = v(x)
    }
    predicate_extensionality(u, v)
}

/// Predicate pullback respects function composition.
theorem predicate_pullback_compose[A, B, C](g: A -> B, f: B -> C, p: C -> Bool) {
    predicate_pullback(compose(f, g), p) = predicate_pullback(g, predicate_pullback(f, p))
} by {
    let u = predicate_pullback(compose(f, g), p)
    let v = predicate_pullback(g, predicate_pullback(f, p))
    forall(x: A) {
        u(x) = p(compose(f, g)(x))
        compose(f, g, x) = f(g(x))
        v(x) = predicate_pullback(f, p, g(x))
        predicate_pullback(f, p, g(x)) = p(f(g(x)))
        u(x) = v(x)
    }
    predicate_extensionality(u, v)
}

/// Predicate pushforward respects function composition.
theorem predicate_pushforward_compose[A, B, C](g: A -> B, f: B -> C, p: A -> Bool) {
    predicate_pushforward(compose(f, g), p) = predicate_pushforward(f, predicate_pushforward(g, p))
} by {
    let u = predicate_pushforward(compose(f, g), p)
    let v = predicate_pushforward(f, predicate_pushforward(g, p))
    forall(z: C) {
        if u(z) {
            predicate_pushforward_has_preimage(compose(f, g), p, z)
            let x: A satisfy {
                compose(f, g)(x) = z and p(x)
            }
            compose(f, g, x) = f(g(x))
            predicate_pushforward_intro(g, p, x)
            predicate_pushforward(g, p, g(x))
            exists(y: B) {
                y = g(x) and f(y) = z and predicate_pushforward(g, p, y)
            }
            predicate_pushforward(f, predicate_pushforward(g, p), z)
            v(z)
        }
        if v(z) {
            predicate_pushforward_has_preimage(f, predicate_pushforward(g, p), z)
            let y: B satisfy {
                f(y) = z and predicate_pushforward(g, p, y)
            }
            predicate_pushforward_has_preimage(g, p, y)
            let x: A satisfy {
                g(x) = y and p(x)
            }
            compose(f, g, x) = f(g(x))
            compose(f, g)(x) = z
            exists(a: A) {
                a = x and compose(f, g)(a) = z and p(a)
            }
            predicate_pushforward(compose(f, g), p, z)
            u(z)
        }
        u(z) = v(z)
    }
    predicate_extensionality(u, v)
}

/// Predicate pullback is monotone with respect to inclusion.
theorem predicate_pullback_monotone[A, B](f: A -> B, p: B -> Bool, q: B -> Bool) {
    predicate_subset(p, q) implies predicate_subset(predicate_pullback(f, p), predicate_pullback(f, q))
} by {
    if predicate_subset(p, q) {
        forall(x: A) {
            if predicate_pullback(f, p, x) {
                predicate_pullback(f, p, x) = p(f(x))
                p(f(x))
                predicate_subset_step(p, q, f(x))
                q(f(x))
                predicate_pullback(f, q, x)
            }
        }
    }
}

/// Predicate pushforward is monotone with respect to inclusion.
theorem predicate_pushforward_monotone[A, B](f: A -> B, p: A -> Bool, q: A -> Bool) {
    predicate_subset(p, q) implies predicate_subset(predicate_pushforward(f, p), predicate_pushforward(f, q))
} by {
    if predicate_subset(p, q) {
        forall(y: B) {
            if predicate_pushforward(f, p, y) {
                predicate_pushforward_has_preimage(f, p, y)
                let x: A satisfy {
                    f(x) = y and p(x)
                }
                predicate_subset_step(p, q, x)
                q(x)
                exists(a: A) {
                    a = x and f(a) = y and q(a)
                }
                predicate_pushforward(f, q, y)
            }
        }
    }
}

/// Pullback after pushforward contains the original predicate.
theorem predicate_subset_pullback_pushforward[A, B](f: A -> B, p: A -> Bool) {
    predicate_subset(p, predicate_pullback(f, predicate_pushforward(f, p)))
} by {
    forall(x: A) {
        if p(x) {
            predicate_pushforward_intro(f, p, x)
            predicate_pushforward(f, p, f(x))
            predicate_pullback(f, predicate_pushforward(f, p), x)
        }
    }
}

/// Pullback after pushforward recovers each original predicate instance along injective functions.
theorem predicate_pullback_pushforward_of_injective_at[A, B](f: A -> B, p: A -> Bool, x: A) {
    is_injective_fn(f) implies predicate_pullback(f, predicate_pushforward(f, p), x) = p(x)
} by {
    if is_injective_fn(f) {
        if predicate_pullback(f, predicate_pushforward(f, p), x) {
            predicate_pullback(f, predicate_pushforward(f, p), x) = predicate_pushforward(f, p, f(x))
            predicate_pushforward_has_preimage(f, p, f(x))
            let a: A satisfy {
                f(a) = f(x) and p(a)
            }
            injective_fn_eq(f, a, x)
            a = x
            p(x)
        }
        if p(x) {
            predicate_subset_pullback_pushforward(f, p)
            predicate_pullback(f, predicate_pushforward(f, p), x)
        }
        predicate_pullback(f, predicate_pushforward(f, p), x) = p(x)
    }
}

/// Pullback after pushforward recovers the original predicate along injective functions.
theorem predicate_pullback_pushforward_of_injective[A, B](f: A -> B, p: A -> Bool) {
    is_injective_fn(f) implies predicate_pullback(f, predicate_pushforward(f, p)) = p
} by {
    let u = predicate_pullback(f, predicate_pushforward(f, p))
    let v = p
    if is_injective_fn(f) {
        forall(x: A) {
            predicate_pullback_pushforward_of_injective_at(f, p, x)
            u(x) = v(x)
        }
        predicate_extensionality(u, v)
    }
}

/// Pushforward after pullback is contained in the target predicate.
theorem predicate_subset_pushforward_pullback[A, B](f: A -> B, p: B -> Bool) {
    predicate_subset(predicate_pushforward(f, predicate_pullback(f, p)), p)
} by {
    forall(y: B) {
        if predicate_pushforward(f, predicate_pullback(f, p), y) {
            predicate_pushforward_has_preimage(f, predicate_pullback(f, p), y)
            let x: A satisfy {
                f(x) = y and predicate_pullback(f, p, x)
            }
            predicate_pullback(f, p, x) = p(f(x))
            p(f(x))
            p(y)
        }
    }
}

/// Pushforward after pullback recovers each target predicate instance along surjective functions.
theorem predicate_pushforward_pullback_of_surjective_at[A, B](f: A -> B, p: B -> Bool, y: B) {
    is_surjective_fn(f) implies predicate_pushforward(f, predicate_pullback(f, p), y) = p(y)
} by {
    if is_surjective_fn(f) {
        if predicate_pushforward(f, predicate_pullback(f, p), y) {
            predicate_subset_pushforward_pullback(f, p)
            p(y)
        }
        if p(y) {
            surjective_fn_has_preimage(f, y)
            let x: A satisfy {
                f(x) = y
            }
            predicate_pullback(f, p, x) = p(f(x))
            predicate_pullback(f, p, x)
            exists(a: A) {
                a = x and f(a) = y and predicate_pullback(f, p, a)
            }
            predicate_pushforward(f, predicate_pullback(f, p), y)
        }
        predicate_pushforward(f, predicate_pullback(f, p), y) = p(y)
    }
}

/// Pushforward after pullback recovers the target predicate along surjective functions.
theorem predicate_pushforward_pullback_of_surjective[A, B](f: A -> B, p: B -> Bool) {
    is_surjective_fn(f) implies predicate_pushforward(f, predicate_pullback(f, p)) = p
} by {
    let u = predicate_pushforward(f, predicate_pullback(f, p))
    let v = p
    if is_surjective_fn(f) {
        forall(y: B) {
            predicate_pushforward_pullback_of_surjective_at(f, p, y)
            u(y) = v(y)
        }
        predicate_extensionality(u, v)
    }
}

/// Bijective pullback after pushforward recovers the original predicate.
theorem predicate_pullback_pushforward_of_bijection[A, B](f: A -> B, p: A -> Bool) {
    is_bijection_fn(f) implies predicate_pullback(f, predicate_pushforward(f, p)) = p
} by {
    if is_bijection_fn(f) {
        bijection_fn_is_injective(f)
        predicate_pullback_pushforward_of_injective(f, p)
        predicate_pullback(f, predicate_pushforward(f, p)) = p
    }
}

/// Bijective pushforward after pullback recovers the target predicate.
theorem predicate_pushforward_pullback_of_bijection[A, B](f: A -> B, p: B -> Bool) {
    is_bijection_fn(f) implies predicate_pushforward(f, predicate_pullback(f, p)) = p
} by {
    if is_bijection_fn(f) {
        bijection_fn_is_surjective(f)
        predicate_pushforward_pullback_of_surjective(f, p)
        predicate_pushforward(f, predicate_pullback(f, p)) = p
    }
}

/// Surjective pullback reflects predicate inclusion.
theorem predicate_pullback_subset_imp_subset_of_surjective[A, B](f: A -> B, p: B -> Bool, q: B -> Bool) {
    is_surjective_fn(f) and predicate_subset(predicate_pullback(f, p), predicate_pullback(f, q)) implies
    predicate_subset(p, q)
} by {
    if is_surjective_fn(f) and predicate_subset(predicate_pullback(f, p), predicate_pullback(f, q)) {
        forall(y: B) {
            if p(y) {
                surjective_fn_has_preimage(f, y)
                let x: A satisfy {
                    f(x) = y
                }
                predicate_pullback(f, p, x) = p(f(x))
                predicate_pullback(f, p, x)
                predicate_subset_step(predicate_pullback(f, p), predicate_pullback(f, q), x)
                predicate_pullback(f, q, x)
                predicate_pullback(f, q, x) = q(f(x))
                q(y)
            }
        }
    }
}

/// Surjective pullback detects predicate inclusion.
theorem predicate_pullback_subset_iff_of_surjective[A, B](f: A -> B, p: B -> Bool, q: B -> Bool) {
    is_surjective_fn(f) implies
    predicate_subset(predicate_pullback(f, p), predicate_pullback(f, q)) = predicate_subset(p, q)
} by {
    if is_surjective_fn(f) {
        if predicate_subset(predicate_pullback(f, p), predicate_pullback(f, q)) {
            predicate_pullback_subset_imp_subset_of_surjective(f, p, q)
            predicate_subset(p, q)
        }
        if predicate_subset(p, q) {
            predicate_pullback_monotone(f, p, q)
            predicate_subset(predicate_pullback(f, p), predicate_pullback(f, q))
        }
        predicate_subset(predicate_pullback(f, p), predicate_pullback(f, q)) = predicate_subset(p, q)
    }
}

/// Surjective pullback reflects predicate equality.
theorem predicate_pullback_eq_imp_eq_of_surjective[A, B](f: A -> B, p: B -> Bool, q: B -> Bool) {
    is_surjective_fn(f) and predicate_pullback(f, p) = predicate_pullback(f, q) implies p = q
} by {
    if is_surjective_fn(f) and predicate_pullback(f, p) = predicate_pullback(f, q) {
        predicate_eq_iff_subset_both(predicate_pullback(f, p), predicate_pullback(f, q))
        predicate_subset(predicate_pullback(f, p), predicate_pullback(f, q))
        predicate_subset(predicate_pullback(f, q), predicate_pullback(f, p))
        predicate_pullback_subset_imp_subset_of_surjective(f, p, q)
        predicate_subset(p, q)
        predicate_pullback_subset_imp_subset_of_surjective(f, q, p)
        predicate_subset(q, p)
        predicate_eq_iff_subset_both(p, q)
        p = q
    }
}

/// Pullback along a surjective function detects predicate equality.
theorem predicate_pullback_eq_iff_of_surjective[A, B](f: A -> B, p: B -> Bool, q: B -> Bool) {
    is_surjective_fn(f) implies (predicate_pullback(f, p) = predicate_pullback(f, q)) = (p = q)
} by {
    if is_surjective_fn(f) {
        if predicate_pullback(f, p) = predicate_pullback(f, q) {
            predicate_pullback_eq_imp_eq_of_surjective(f, p, q)
            p = q
        }
        if p = q {
            predicate_pullback(f, p) = predicate_pullback(f, q)
        }
        (predicate_pullback(f, p) = predicate_pullback(f, q)) = (p = q)
    }
}

/// Injective pushforward reflects predicate inclusion.
theorem predicate_pushforward_subset_imp_subset_of_injective[A, B](f: A -> B, p: A -> Bool, q: A -> Bool) {
    is_injective_fn(f) and predicate_subset(predicate_pushforward(f, p), predicate_pushforward(f, q)) implies
    predicate_subset(p, q)
} by {
    if is_injective_fn(f) and predicate_subset(predicate_pushforward(f, p), predicate_pushforward(f, q)) {
        forall(x: A) {
            if p(x) {
                predicate_pushforward_intro(f, p, x)
                predicate_pushforward(f, p, f(x))
                predicate_subset_step(predicate_pushforward(f, p), predicate_pushforward(f, q), f(x))
                predicate_pushforward(f, q, f(x))
                predicate_pushforward_has_preimage(f, q, f(x))
                let a: A satisfy {
                    f(a) = f(x) and q(a)
                }
                injective_fn_eq(f, a, x)
                a = x
                q(x)
            }
        }
    }
}

/// Injective pushforward detects predicate inclusion.
theorem predicate_pushforward_subset_iff_of_injective[A, B](f: A -> B, p: A -> Bool, q: A -> Bool) {
    is_injective_fn(f) implies
    predicate_subset(predicate_pushforward(f, p), predicate_pushforward(f, q)) = predicate_subset(p, q)
} by {
    if is_injective_fn(f) {
        if predicate_subset(predicate_pushforward(f, p), predicate_pushforward(f, q)) {
            predicate_pushforward_subset_imp_subset_of_injective(f, p, q)
            predicate_subset(p, q)
        }
        if predicate_subset(p, q) {
            predicate_pushforward_monotone(f, p, q)
            predicate_subset(predicate_pushforward(f, p), predicate_pushforward(f, q))
        }
        predicate_subset(predicate_pushforward(f, p), predicate_pushforward(f, q)) = predicate_subset(p, q)
    }
}

/// Injective pushforward reflects predicate equality.
theorem predicate_pushforward_eq_imp_eq_of_injective[A, B](f: A -> B, p: A -> Bool, q: A -> Bool) {
    is_injective_fn(f) and predicate_pushforward(f, p) = predicate_pushforward(f, q) implies p = q
} by {
    if is_injective_fn(f) and predicate_pushforward(f, p) = predicate_pushforward(f, q) {
        predicate_eq_iff_subset_both(predicate_pushforward(f, p), predicate_pushforward(f, q))
        predicate_subset(predicate_pushforward(f, p), predicate_pushforward(f, q))
        predicate_subset(predicate_pushforward(f, q), predicate_pushforward(f, p))
        predicate_pushforward_subset_imp_subset_of_injective(f, p, q)
        predicate_subset(p, q)
        predicate_pushforward_subset_imp_subset_of_injective(f, q, p)
        predicate_subset(q, p)
        predicate_eq_iff_subset_both(p, q)
        p = q
    }
}

/// Pushforward along an injective function detects predicate equality.
theorem predicate_pushforward_eq_iff_of_injective[A, B](f: A -> B, p: A -> Bool, q: A -> Bool) {
    is_injective_fn(f) implies (predicate_pushforward(f, p) = predicate_pushforward(f, q)) = (p = q)
} by {
    if is_injective_fn(f) {
        if predicate_pushforward(f, p) = predicate_pushforward(f, q) {
            predicate_pushforward_eq_imp_eq_of_injective(f, p, q)
            p = q
        }
        if p = q {
            predicate_pushforward(f, p) = predicate_pushforward(f, q)
        }
        (predicate_pushforward(f, p) = predicate_pushforward(f, q)) = (p = q)
    }
}

/// Bijective pullback detects predicate inclusion.
theorem predicate_pullback_subset_iff_of_bijection[A, B](f: A -> B, p: B -> Bool, q: B -> Bool) {
    is_bijection_fn(f) implies
    predicate_subset(predicate_pullback(f, p), predicate_pullback(f, q)) = predicate_subset(p, q)
} by {
    if is_bijection_fn(f) {
        bijection_fn_is_surjective(f)
        predicate_pullback_subset_iff_of_surjective(f, p, q)
        predicate_subset(predicate_pullback(f, p), predicate_pullback(f, q)) = predicate_subset(p, q)
    }
}

/// Bijective pushforward detects predicate inclusion.
theorem predicate_pushforward_subset_iff_of_bijection[A, B](f: A -> B, p: A -> Bool, q: A -> Bool) {
    is_bijection_fn(f) implies
    predicate_subset(predicate_pushforward(f, p), predicate_pushforward(f, q)) = predicate_subset(p, q)
} by {
    if is_bijection_fn(f) {
        bijection_fn_is_injective(f)
        predicate_pushforward_subset_iff_of_injective(f, p, q)
        predicate_subset(predicate_pushforward(f, p), predicate_pushforward(f, q)) = predicate_subset(p, q)
    }
}

/// Bijective pullback detects predicate equality.
theorem predicate_pullback_eq_iff_of_bijection[A, B](f: A -> B, p: B -> Bool, q: B -> Bool) {
    is_bijection_fn(f) implies (predicate_pullback(f, p) = predicate_pullback(f, q)) = (p = q)
} by {
    if is_bijection_fn(f) {
        bijection_fn_is_surjective(f)
        predicate_pullback_eq_iff_of_surjective(f, p, q)
        (predicate_pullback(f, p) = predicate_pullback(f, q)) = (p = q)
    }
}

/// Bijective pushforward detects predicate equality.
theorem predicate_pushforward_eq_iff_of_bijection[A, B](f: A -> B, p: A -> Bool, q: A -> Bool) {
    is_bijection_fn(f) implies (predicate_pushforward(f, p) = predicate_pushforward(f, q)) = (p = q)
} by {
    if is_bijection_fn(f) {
        bijection_fn_is_injective(f)
        predicate_pushforward_eq_iff_of_injective(f, p, q)
        (predicate_pushforward(f, p) = predicate_pushforward(f, q)) = (p = q)
    }
}

/// Pullback after pushforward recovers a predicate along a bundled bijection.
theorem predicate_pullback_pushforward_of_bijection_map[A, B](e: Bijection[A, B], p: A -> Bool) {
    predicate_pullback(e.map, predicate_pushforward(e.map, p)) = p
} by {
    bijection_map_is_bijection(e)
    predicate_pullback_pushforward_of_bijection(e.map, p)
}

/// Pushforward after pullback recovers a predicate along a bundled bijection.
theorem predicate_pushforward_pullback_of_bijection_map[A, B](e: Bijection[A, B], p: B -> Bool) {
    predicate_pushforward(e.map, predicate_pullback(e.map, p)) = p
} by {
    bijection_map_is_bijection(e)
    predicate_pushforward_pullback_of_bijection(e.map, p)
}

/// Bundled bijective pullback detects predicate inclusion.
theorem predicate_pullback_subset_iff_of_bijection_map[A, B](e: Bijection[A, B], p: B -> Bool, q: B -> Bool) {
    predicate_subset(predicate_pullback(e.map, p), predicate_pullback(e.map, q)) = predicate_subset(p, q)
} by {
    bijection_map_is_bijection(e)
    predicate_pullback_subset_iff_of_bijection(e.map, p, q)
}

/// Bundled bijective pushforward detects predicate inclusion.
theorem predicate_pushforward_subset_iff_of_bijection_map[A, B](e: Bijection[A, B], p: A -> Bool, q: A -> Bool) {
    predicate_subset(predicate_pushforward(e.map, p), predicate_pushforward(e.map, q)) = predicate_subset(p, q)
} by {
    bijection_map_is_bijection(e)
    predicate_pushforward_subset_iff_of_bijection(e.map, p, q)
}

/// Bundled bijective pullback detects predicate equality.
theorem predicate_pullback_eq_iff_of_bijection_map[A, B](e: Bijection[A, B], p: B -> Bool, q: B -> Bool) {
    (predicate_pullback(e.map, p) = predicate_pullback(e.map, q)) = (p = q)
} by {
    bijection_map_is_bijection(e)
    predicate_pullback_eq_iff_of_bijection(e.map, p, q)
}

/// Bundled bijective pushforward detects predicate equality.
theorem predicate_pushforward_eq_iff_of_bijection_map[A, B](e: Bijection[A, B], p: A -> Bool, q: A -> Bool) {
    (predicate_pushforward(e.map, p) = predicate_pushforward(e.map, q)) = (p = q)
} by {
    bijection_map_is_bijection(e)
    predicate_pushforward_eq_iff_of_bijection(e.map, p, q)
}

/// Related inputs give a related image pair.
theorem relation_pushforward_intro[A, B](f: A -> B, r: (A, A) -> Bool, x: A, y: A) {
    r(x, y) implies relation_pushforward(f, r, f(x), f(y))
} by {
    if r(x, y) {
        exists(a: A, b: A) {
            a = x and b = y and f(a) = f(x) and f(b) = f(y) and r(a, b)
        }
        relation_pushforward(f, r, f(x), f(y))
    }
}

/// A pushed-forward relation instance has related preimages.
theorem relation_pushforward_has_preimages[A, B](f: A -> B, r: (A, A) -> Bool, u: B, v: B) {
    relation_pushforward(f, r, u, v) implies exists(x: A, y: A) {
        f(x) = u and f(y) = v and r(x, y)
    }
} by {
    if relation_pushforward(f, r, u, v) {
        exists(x: A, y: A) {
            f(x) = u and f(y) = v and r(x, y)
        }
    }
}

/// Pushforward is monotone with respect to relation inclusion.
theorem relation_pushforward_monotone[A, B](f: A -> B, r: (A, A) -> Bool, s: (A, A) -> Bool) {
    relation_subset(r, s) implies relation_subset(relation_pushforward(f, r), relation_pushforward(f, s))
} by {
    if relation_subset(r, s) {
        forall(u: B, v: B) {
            if relation_pushforward(f, r, u, v) {
                relation_pushforward_has_preimages(f, r, u, v)
                let x: A satisfy {
                    exists(y: A) {
                        f(x) = u and f(y) = v and r(x, y)
                    }
                }
                let y: A satisfy {
                    f(x) = u and f(y) = v and r(x, y)
                }
                relation_subset_step(r, s, x, y)
                s(x, y)
                exists(a: A, b: A) {
                    a = x and b = y and f(a) = u and f(b) = v and s(a, b)
                }
                relation_pushforward(f, s, u, v)
            }
        }
    }
}

/// Pushing forward along the identity function leaves a relation unchanged.
theorem relation_pushforward_identity[T](r: (T, T) -> Bool) {
    relation_pushforward(identity_fn[T], r) = r
} by {
    let u = relation_pushforward(identity_fn[T], r)
    let v = r
    forall(x: T, y: T) {
        if u(x, y) {
            relation_pushforward_has_preimages(identity_fn[T], r, x, y)
            let a: T satisfy {
                exists(b: T) {
                    identity_fn[T](a) = x and identity_fn[T](b) = y and r(a, b)
                }
            }
            let b: T satisfy {
                identity_fn[T](a) = x and identity_fn[T](b) = y and r(a, b)
            }
            identity_fn[T](a) = a
            identity_fn[T](b) = b
            a = x
            b = y
            v(x, y)
        }
        if v(x, y) {
            relation_pushforward_intro(identity_fn[T], r, x, y)
            identity_fn[T](x) = x
            identity_fn[T](y) = y
            u(x, y)
        }
        u(x, y) = v(x, y)
    }
    binary_function_extensionality(u, v)
}

/// Pushing forward along a composite is the same as pushing forward twice.
theorem relation_pushforward_compose[A, B, C](g: A -> B, f: B -> C, r: (A, A) -> Bool) {
    relation_pushforward(compose(f, g), r) = relation_pushforward(f, relation_pushforward(g, r))
} by {
    let u = relation_pushforward(compose(f, g), r)
    let v = relation_pushforward(f, relation_pushforward(g, r))
    forall(p: C, q: C) {
        if u(p, q) {
            relation_pushforward_has_preimages(compose(f, g), r, p, q)
            let x: A satisfy {
                exists(y: A) {
                    compose(f, g)(x) = p and compose(f, g)(y) = q and r(x, y)
                }
            }
            let y: A satisfy {
                compose(f, g)(x) = p and compose(f, g)(y) = q and r(x, y)
            }
            compose(f, g, x) = f(g(x))
            compose(f, g, y) = f(g(y))
            f(g(x)) = p
            f(g(y)) = q
            relation_pushforward_intro(g, r, x, y)
            relation_pushforward(g, r, g(x), g(y))
            exists(a: B, b: B) {
                a = g(x) and b = g(y) and f(a) = p and f(b) = q and relation_pushforward(g, r, a, b)
            }
            relation_pushforward(f, relation_pushforward(g, r), p, q)
            v(p, q)
        }
        if v(p, q) {
            relation_pushforward_has_preimages(f, relation_pushforward(g, r), p, q)
            let a: B satisfy {
                exists(b: B) {
                    f(a) = p and f(b) = q and relation_pushforward(g, r, a, b)
                }
            }
            let b: B satisfy {
                f(a) = p and f(b) = q and relation_pushforward(g, r, a, b)
            }
            relation_pushforward_has_preimages(g, r, a, b)
            let x: A satisfy {
                exists(y: A) {
                    g(x) = a and g(y) = b and r(x, y)
                }
            }
            let y: A satisfy {
                g(x) = a and g(y) = b and r(x, y)
            }
            compose(f, g, x) = f(g(x))
            compose(f, g, y) = f(g(y))
            compose(f, g)(x) = p
            compose(f, g)(y) = q
            f(g(x)) = p
            f(g(y)) = q
            exists(x0: A, y0: A) {
                x0 = x and y0 = y and compose(f, g)(x0) = p and compose(f, g)(y0) = q and r(x0, y0)
            }
            relation_pushforward(compose(f, g), r, p, q)
            u(p, q)
        }
        u(p, q) = v(p, q)
    }
    binary_function_extensionality(u, v)
}

/// Pushforward preserves reflexive relations along surjective functions.
theorem relation_pushforward_is_reflexive_of_surjective[A, B](f: A -> B, r: (A, A) -> Bool) {
    is_surjective_fn(f) and is_reflexive(r) implies is_reflexive(relation_pushforward(f, r))
} by {
    if is_surjective_fn(f) and is_reflexive(r) {
        forall(u: B) {
            surjective_fn_has_preimage(f, u)
            let x: A satisfy {
                f(x) = u
            }
            reflexive_self(r, x)
            relation_pushforward_intro(f, r, x, x)
            relation_pushforward(f, r, u, u)
        }
    }
}

/// Pushforward preserves symmetric relations.
theorem relation_pushforward_is_symmetric[A, B](f: A -> B, r: (A, A) -> Bool) {
    is_symmetric(r) implies is_symmetric(relation_pushforward(f, r))
} by {
    if is_symmetric(r) {
        forall(u: B, v: B) {
            if relation_pushforward(f, r, u, v) {
                relation_pushforward_has_preimages(f, r, u, v)
                let x: A satisfy {
                    exists(y: A) {
                        f(x) = u and f(y) = v and r(x, y)
                    }
                }
                let y: A satisfy {
                    f(x) = u and f(y) = v and r(x, y)
                }
                symmetric_flip(r, x, y)
                r(y, x)
                exists(a: A, b: A) {
                    a = y and b = x and f(a) = v and f(b) = u and r(a, b)
                }
                relation_pushforward(f, r, v, u)
            }
        }
    }
}

/// Pushforward preserves transitive relations along injective functions.
theorem relation_pushforward_is_transitive_of_injective[A, B](f: A -> B, r: (A, A) -> Bool) {
    is_injective_fn(f) and is_transitive(r) implies is_transitive(relation_pushforward(f, r))
} by {
    if is_injective_fn(f) and is_transitive(r) {
        forall(u: B, v: B, w: B) {
            if relation_pushforward(f, r, u, v) and relation_pushforward(f, r, v, w) {
                relation_pushforward_has_preimages(f, r, u, v)
                let x1: A satisfy {
                    exists(y1: A) {
                        f(x1) = u and f(y1) = v and r(x1, y1)
                    }
                }
                let y1: A satisfy {
                    f(x1) = u and f(y1) = v and r(x1, y1)
                }
                relation_pushforward_has_preimages(f, r, v, w)
                let x2: A satisfy {
                    exists(y2: A) {
                        f(x2) = v and f(y2) = w and r(x2, y2)
                    }
                }
                let y2: A satisfy {
                    f(x2) = v and f(y2) = w and r(x2, y2)
                }
                f(y1) = f(x2)
                injective_fn_eq(f, y1, x2)
                y1 = x2
                r(x1, x2)
                transitive_step(r, x1, x2, y2)
                r(x1, y2)
                exists(a: A, b: A) {
                    a = x1 and b = y2 and f(a) = u and f(b) = w and r(a, b)
                }
                relation_pushforward(f, r, u, w)
            }
        }
    }
}

/// Pushforward preserves total relations along surjective functions.
theorem relation_pushforward_is_total_of_surjective[A, B](f: A -> B, r: (A, A) -> Bool) {
    is_surjective_fn(f) and is_total(r) implies is_total(relation_pushforward(f, r))
} by {
    if is_surjective_fn(f) and is_total(r) {
        is_total(r) = forall(x0: A, y0: A) {
            r(x0, y0) or r(y0, x0)
        }
        forall(u: B, v: B) {
            surjective_fn_has_preimage(f, u)
            let x: A satisfy {
                f(x) = u
            }
            surjective_fn_has_preimage(f, v)
            let y: A satisfy {
                f(y) = v
            }
            r(x, y) or r(y, x)
            if r(x, y) {
                relation_pushforward_intro(f, r, x, y)
                relation_pushforward(f, r, u, v)
                relation_pushforward(f, r, u, v) or relation_pushforward(f, r, v, u)
            } else {
                relation_pushforward_intro(f, r, y, x)
                relation_pushforward(f, r, v, u)
                relation_pushforward(f, r, u, v) or relation_pushforward(f, r, v, u)
            }
        }
    }
}

/// Pushforward preserves antisymmetric relations along injective functions.
theorem relation_pushforward_is_antisymmetric_of_injective[A, B](f: A -> B, r: (A, A) -> Bool) {
    is_injective_fn(f) and is_antisymmetric(r) implies is_antisymmetric(relation_pushforward(f, r))
} by {
    if is_injective_fn(f) and is_antisymmetric(r) {
        forall(u: B, v: B) {
            if relation_pushforward(f, r, u, v) and relation_pushforward(f, r, v, u) {
                relation_pushforward_has_preimages(f, r, u, v)
                let x1: A satisfy {
                    exists(y1: A) {
                        f(x1) = u and f(y1) = v and r(x1, y1)
                    }
                }
                let y1: A satisfy {
                    f(x1) = u and f(y1) = v and r(x1, y1)
                }
                relation_pushforward_has_preimages(f, r, v, u)
                let x2: A satisfy {
                    exists(y2: A) {
                        f(x2) = v and f(y2) = u and r(x2, y2)
                    }
                }
                let y2: A satisfy {
                    f(x2) = v and f(y2) = u and r(x2, y2)
                }
                f(y1) = f(x2)
                injective_fn_eq(f, y1, x2)
                y1 = x2
                f(x1) = f(y2)
                injective_fn_eq(f, x1, y2)
                x1 = y2
                r(y1, x1)
                antisymmetric_eq(r, x1, y1)
                x1 = y1
                f(x1) = f(y1)
                u = v
            }
        }
    }
}

/// Pushforward preserves equivalence relations along bijective functions.
theorem relation_pushforward_is_equivalence_of_bijection[A, B](f: A -> B, r: (A, A) -> Bool) {
    is_bijection_fn(f) and is_equivalence(r) implies is_equivalence(relation_pushforward(f, r))
} by {
    if is_bijection_fn(f) and is_equivalence(r) {
        bijection_fn_is_surjective(f)
        is_surjective_fn(f)
        equivalence_is_reflexive(r)
        is_reflexive(r)
        relation_pushforward_is_reflexive_of_surjective(f, r)
        is_reflexive(relation_pushforward(f, r))
        equivalence_is_symmetric(r)
        is_symmetric(r)
        relation_pushforward_is_symmetric(f, r)
        is_symmetric(relation_pushforward(f, r))
        bijection_fn_is_injective(f)
        is_injective_fn(f)
        equivalence_is_transitive(r)
        is_transitive(r)
        relation_pushforward_is_transitive_of_injective(f, r)
        is_transitive(relation_pushforward(f, r))
        is_equivalence(relation_pushforward(f, r))
    }
}

/// Pushforward preserves reflexive, transitive, and antisymmetric order data along bijective functions.
theorem relation_pushforward_is_reflexive_transitive_antisymmetric_of_bijection[A, B](f: A -> B, r: (A, A) -> Bool) {
    is_bijection_fn(f) and is_reflexive(r) and is_transitive(r) and is_antisymmetric(r) implies
    is_reflexive(relation_pushforward(f, r)) and
    is_transitive(relation_pushforward(f, r)) and
    is_antisymmetric(relation_pushforward(f, r))
} by {
    if is_bijection_fn(f) and is_reflexive(r) and is_transitive(r) and is_antisymmetric(r) {
        bijection_fn_is_surjective(f)
        is_surjective_fn(f)
        relation_pushforward_is_reflexive_of_surjective(f, r)
        is_reflexive(relation_pushforward(f, r))
        bijection_fn_is_injective(f)
        is_injective_fn(f)
        relation_pushforward_is_transitive_of_injective(f, r)
        is_transitive(relation_pushforward(f, r))
        relation_pushforward_is_antisymmetric_of_injective(f, r)
        is_antisymmetric(relation_pushforward(f, r))
        is_reflexive(relation_pushforward(f, r)) and
        is_transitive(relation_pushforward(f, r)) and
        is_antisymmetric(relation_pushforward(f, r))
    }
}

/// Pushforward preserves equivalence relations along a bundled bijection.
theorem relation_pushforward_is_equivalence_of_bijection_map[A, B](e: Bijection[A, B], r: (A, A) -> Bool) {
    is_equivalence(r) implies is_equivalence(relation_pushforward(e.map, r))
} by {
    if is_equivalence(r) {
        bijection_map_is_bijection(e)
        relation_pushforward_is_equivalence_of_bijection(e.map, r)
        is_equivalence(relation_pushforward(e.map, r))
    }
}

/// Pushforward preserves order data along a bundled bijection.
theorem relation_pushforward_is_reflexive_transitive_antisymmetric_of_bijection_map[A, B](
    e: Bijection[A, B],
    r: (A, A) -> Bool
) {
    is_reflexive(r) and is_transitive(r) and is_antisymmetric(r) implies
    is_reflexive(relation_pushforward(e.map, r)) and
    is_transitive(relation_pushforward(e.map, r)) and
    is_antisymmetric(relation_pushforward(e.map, r))
} by {
    if is_reflexive(r) and is_transitive(r) and is_antisymmetric(r) {
        bijection_map_is_bijection(e)
        relation_pushforward_is_reflexive_transitive_antisymmetric_of_bijection(e.map, r)
        is_reflexive(relation_pushforward(e.map, r)) and
        is_transitive(relation_pushforward(e.map, r)) and
        is_antisymmetric(relation_pushforward(e.map, r))
    }
}

/// Pullback after pushforward contains the original relation.
theorem relation_subset_pullback_pushforward[A, B](f: A -> B, r: (A, A) -> Bool) {
    relation_subset(r, relation_pullback(f, relation_pushforward(f, r)))
} by {
    forall(x: A, y: A) {
        if r(x, y) {
            relation_pushforward_intro(f, r, x, y)
            relation_pullback(f, relation_pushforward(f, r), x, y)
        }
    }
}

/// Pullback after pushforward recovers each original relation instance along injective functions.
theorem relation_pullback_pushforward_of_injective_at[A, B](f: A -> B, r: (A, A) -> Bool, x: A, y: A) {
    is_injective_fn(f) implies relation_pullback(f, relation_pushforward(f, r), x, y) = r(x, y)
} by {
    if is_injective_fn(f) {
        if relation_pullback(f, relation_pushforward(f, r), x, y) {
            relation_pullback(f, relation_pushforward(f, r), x, y) =
                relation_pushforward(f, r, f(x), f(y))
            relation_pushforward_has_preimages(f, r, f(x), f(y))
            let a: A satisfy {
                exists(b: A) {
                    f(a) = f(x) and f(b) = f(y) and r(a, b)
                }
            }
            let b: A satisfy {
                f(a) = f(x) and f(b) = f(y) and r(a, b)
            }
            injective_fn_eq(f, a, x)
            a = x
            injective_fn_eq(f, b, y)
            b = y
            r(x, y)
        }
        if r(x, y) {
            relation_subset_pullback_pushforward(f, r)
            relation_pullback(f, relation_pushforward(f, r), x, y)
        }
        relation_pullback(f, relation_pushforward(f, r), x, y) = r(x, y)
    }
}

/// Pullback after pushforward recovers the original relation along injective functions.
theorem relation_pullback_pushforward_of_injective[A, B](f: A -> B, r: (A, A) -> Bool) {
    is_injective_fn(f) implies relation_pullback(f, relation_pushforward(f, r)) = r
} by {
    let u = relation_pullback(f, relation_pushforward(f, r))
    let v = r
    if is_injective_fn(f) {
        forall(x: A, y: A) {
            relation_pullback_pushforward_of_injective_at(f, r, x, y)
            u(x, y) = v(x, y)
        }
        binary_function_extensionality(u, v)
    }
}

/// Pushforward after pullback is contained in the target relation.
theorem relation_subset_pushforward_pullback[A, B](f: A -> B, r: (B, B) -> Bool) {
    relation_subset(relation_pushforward(f, relation_pullback(f, r)), r)
} by {
    forall(u: B, v: B) {
        if relation_pushforward(f, relation_pullback(f, r), u, v) {
            relation_pushforward_has_preimages(f, relation_pullback(f, r), u, v)
            let x: A satisfy {
                exists(y: A) {
                    f(x) = u and f(y) = v and relation_pullback(f, r, x, y)
                }
            }
            let y: A satisfy {
                f(x) = u and f(y) = v and relation_pullback(f, r, x, y)
            }
            relation_pullback(f, r, x, y) = r(f(x), f(y))
            r(f(x), f(y))
            r(u, v)
        }
    }
}

/// Pushforward after pullback recovers each target relation instance along surjective functions.
theorem relation_pushforward_pullback_of_surjective_at[A, B](f: A -> B, r: (B, B) -> Bool, p: B, q: B) {
    is_surjective_fn(f) implies relation_pushforward(f, relation_pullback(f, r), p, q) = r(p, q)
} by {
    if is_surjective_fn(f) {
        if relation_pushforward(f, relation_pullback(f, r), p, q) {
            relation_subset_pushforward_pullback(f, r)
            r(p, q)
        }
        if r(p, q) {
            surjective_fn_has_preimage(f, p)
            let x: A satisfy {
                f(x) = p
            }
            surjective_fn_has_preimage(f, q)
            let y: A satisfy {
                f(y) = q
            }
            relation_pullback(f, r, x, y) = r(f(x), f(y))
            relation_pullback(f, r, x, y)
            exists(a: A, b: A) {
                a = x and b = y and f(a) = p and f(b) = q and relation_pullback(f, r, a, b)
            }
            relation_pushforward(f, relation_pullback(f, r), p, q)
        }
        relation_pushforward(f, relation_pullback(f, r), p, q) = r(p, q)
    }
}

/// Pushforward after pullback recovers the target relation along surjective functions.
theorem relation_pushforward_pullback_of_surjective[A, B](f: A -> B, r: (B, B) -> Bool) {
    is_surjective_fn(f) implies relation_pushforward(f, relation_pullback(f, r)) = r
} by {
    let u = relation_pushforward(f, relation_pullback(f, r))
    let v = r
    if is_surjective_fn(f) {
        forall(p: B, q: B) {
            relation_pushforward_pullback_of_surjective_at(f, r, p, q)
            u(p, q) = v(p, q)
        }
        binary_function_extensionality(u, v)
    }
}

/// Pullback after pushforward recovers a relation along a bundled bijection.
theorem relation_pullback_pushforward_of_bijection_map[A, B](e: Bijection[A, B], r: (A, A) -> Bool) {
    relation_pullback(e.map, relation_pushforward(e.map, r)) = r
} by {
    bijection_map_is_injective(e)
    relation_pullback_pushforward_of_injective(e.map, r)
}

/// Pushforward after pullback recovers a relation along a bundled bijection.
theorem relation_pushforward_pullback_of_bijection_map[A, B](e: Bijection[A, B], r: (B, B) -> Bool) {
    relation_pushforward(e.map, relation_pullback(e.map, r)) = r
} by {
    bijection_map_is_surjective(e)
    relation_pushforward_pullback_of_surjective(e.map, r)
}

/// Pullback commutes with converse.
theorem relation_pullback_converse[A, B](f: A -> B, r: (B, B) -> Bool) {
    relation_pullback(f, relation_converse(r)) = relation_converse(relation_pullback(f, r))
} by {
    let u = relation_pullback(f, relation_converse(r))
    let v = relation_converse(relation_pullback(f, r))
    forall(x: A, y: A) {
        u(x, y) = relation_converse(r, f(x), f(y))
        relation_converse(r, f(x), f(y)) = r(f(y), f(x))
        v(x, y) = relation_pullback(f, r, y, x)
        relation_pullback(f, r, y, x) = r(f(y), f(x))
        u(x, y) = v(x, y)
    }
    binary_function_extensionality(u, v)
}

/// Pullback commutes with symmetric closure.
theorem relation_pullback_symmetric_closure[A, B](f: A -> B, r: (B, B) -> Bool) {
    relation_pullback(f, relation_symmetric_closure(r)) = relation_symmetric_closure(relation_pullback(f, r))
} by {
    let u = relation_pullback(f, relation_symmetric_closure(r))
    let v = relation_symmetric_closure(relation_pullback(f, r))
    forall(x: A, y: A) {
        u(x, y) = relation_symmetric_closure(r, f(x), f(y))
        relation_symmetric_closure(r, f(x), f(y)) = (r(f(x), f(y)) or relation_converse(r, f(x), f(y)))
        relation_pullback(f, r, x, y) = r(f(x), f(y))
        relation_pullback_converse(f, r)
        relation_converse(relation_pullback(f, r), x, y) = relation_pullback(f, relation_converse(r), x, y)
        relation_pullback(f, relation_converse(r), x, y) = relation_converse(r, f(x), f(y))
        v(x, y) = (relation_pullback(f, r, x, y) or relation_converse(relation_pullback(f, r), x, y))
        u(x, y) = v(x, y)
    }
    binary_function_extensionality(u, v)
}

/// True if `f` sends equivalent inputs to equivalent outputs.
define respects_equivalence[T, U](f: T -> U, r: (T, T) -> Bool, s: (U, U) -> Bool) -> Bool {
    forall(x1: T, x2: T) {
        r(x1, x2) implies s(f(x1), f(x2))
    }
}

/// True if a function has equal values on related inputs.
define respects_function[T, U](r: (T, T) -> Bool, f: T -> U) -> Bool {
    forall(x: T, y: T) {
        r(x, y) implies f(x) = f(y)
    }
}

/// True if a predicate has the same truth value on related inputs.
define respects_predicate[T](r: (T, T) -> Bool, p: T -> Bool) -> Bool {
    respects_function(r, p)
}

/// The function on a target domain induced by a bijection from a source domain.
define function_pushforward[A: Inhabited, B, C](f: A -> B, g: A -> C, y: B) -> C {
    g(inverse_fn(f, y))
}

/// Transporting a function across a bijection agrees with the original function on images.
theorem function_pushforward_apply_image_of_bijection[A: Inhabited, B, C](f: A -> B, g: A -> C, x: A) {
    is_bijection_fn(f) implies function_pushforward(f, g, f(x)) = g(x)
} by {
    if is_bijection_fn(f) {
        function_pushforward(f, g, f(x)) = g(inverse_fn(f, f(x)))
        inverse_fn_apply_image_of_bijection(f, x)
        inverse_fn(f, f(x)) = x
        function_pushforward(f, g, f(x)) = g(x)
    }
}

/// Transporting a function across a surjection applies through a chosen preimage.
theorem function_pushforward_apply_of_surjective[A: Inhabited, B, C](f: A -> B, g: A -> C, y: B) {
    is_surjective_fn(f) implies exists(x: A) {
        f(x) = y and function_pushforward(f, g, y) = g(x)
    }
} by {
    if is_surjective_fn(f) {
        let x = inverse_fn(f, y)
        inverse_fn_apply_of_surjective(f, y)
        f(x) = y
        function_pushforward(f, g, y) = g(x)
        exists(a: A) {
            a = x and f(a) = y and function_pushforward(f, g, y) = g(a)
        }
    }
}

/// Transporting a relation-respecting function across a bijection preserves respect for pushed-forward relations.
theorem function_pushforward_respects_equivalence_of_bijection[A: Inhabited, B, C](
    f: A -> B,
    g: A -> C,
    r: (A, A) -> Bool,
    s: (C, C) -> Bool
) {
    is_bijection_fn(f) and respects_equivalence(g, r, s) implies
    respects_equivalence(function_pushforward(f, g), relation_pushforward(f, r), s)
} by {
    if is_bijection_fn(f) and respects_equivalence(g, r, s) {
        bijection_fn_is_injective(f)
        is_injective_fn(f)
        forall(u: B, v: B) {
            if relation_pushforward(f, r, u, v) {
                relation_pushforward_has_preimages(f, r, u, v)
                let x: A satisfy {
                    exists(y: A) {
                        f(x) = u and f(y) = v and r(x, y)
                    }
                }
                let y: A satisfy {
                    f(x) = u and f(y) = v and r(x, y)
                }
                bijection_fn_is_surjective(f)
                is_surjective_fn(f)
                inverse_fn_apply_of_surjective(f, u)
                f(inverse_fn(f, u)) = u
                f(x) = u
                f(inverse_fn(f, u)) = f(x)
                injective_fn_eq(f, inverse_fn(f, u), x)
                inverse_fn(f, u) = x
                inverse_fn_apply_of_surjective(f, v)
                f(inverse_fn(f, v)) = v
                f(y) = v
                f(inverse_fn(f, v)) = f(y)
                injective_fn_eq(f, inverse_fn(f, v), y)
                inverse_fn(f, v) = y
                function_pushforward(f, g, u) = g(inverse_fn(f, u))
                function_pushforward(f, g, v) = g(inverse_fn(f, v))
                function_pushforward(f, g, u) = g(x)
                function_pushforward(f, g, v) = g(y)
                respects_equivalence(g, r, s) = forall(a: A, b: A) {
                    r(a, b) implies s(g(a), g(b))
                }
                s(g(x), g(y))
                s(function_pushforward(f, g, u), function_pushforward(f, g, v))
            }
        }
        respects_equivalence(function_pushforward(f, g), relation_pushforward(f, r), s)
    }
}

/// Respecting a relation is the same as landing inside the pullback relation.
theorem respects_equivalence_eq_relation_subset_pullback[T, U](f: T -> U, r: (T, T) -> Bool, s: (U, U) -> Bool) {
    respects_equivalence(f, r, s) = relation_subset(r, relation_pullback(f, s))
} by {
    if respects_equivalence(f, r, s) {
        respects_equivalence(f, r, s) = forall(a: T, b: T) {
            r(a, b) implies s(f(a), f(b))
        }
        forall(x1: T, x2: T) {
            if r(x1, x2) {
                s(f(x1), f(x2))
                relation_pullback(f, s, x1, x2)
            }
        }
        relation_subset(r, relation_pullback(f, s))
    }
    if relation_subset(r, relation_pullback(f, s)) {
        relation_subset(r, relation_pullback(f, s)) = forall(a: T, b: T) {
            r(a, b) implies relation_pullback(f, s, a, b)
        }
        forall(x1: T, x2: T) {
            if r(x1, x2) {
                relation_subset_step(r, relation_pullback(f, s), x1, x2)
                relation_pullback(f, s, x1, x2)
                s(f(x1), f(x2))
            }
        }
        respects_equivalence(f, r, s)
    }
    respects_equivalence(f, r, s) = relation_subset(r, relation_pullback(f, s))
}

/// Function invariance is relation preservation into equality.
theorem respects_function_eq_respects_equivalence_eq_relation[T, U](r: (T, T) -> Bool, f: T -> U) {
    respects_function(r, f) = respects_equivalence(f, r, eq_relation[U])
} by {
    if respects_function(r, f) {
        respects_function(r, f) = forall(x0: T, y0: T) {
            r(x0, y0) implies f(x0) = f(y0)
        }
        forall(x: T, y: T) {
            if r(x, y) {
                f(x) = f(y)
                eq_relation[U](f(x), f(y))
            }
        }
        respects_equivalence(f, r, eq_relation[U])
    }
    if respects_equivalence(f, r, eq_relation[U]) {
        respects_equivalence(f, r, eq_relation[U]) = forall(x0: T, y0: T) {
            r(x0, y0) implies eq_relation[U](f(x0), f(y0))
        }
        forall(x: T, y: T) {
            if r(x, y) {
                eq_relation[U](f(x), f(y))
                f(x) = f(y)
            }
        }
        respects_function(r, f)
    }
    respects_function(r, f) = respects_equivalence(f, r, eq_relation[U])
}

/// A relation-invariant function has equal values at any related pair.
theorem respects_function_step[T, U](r: (T, T) -> Bool, f: T -> U, x: T, y: T) {
    respects_function(r, f) and r(x, y) implies f(x) = f(y)
} by {
    if respects_function(r, f) and r(x, y) {
        respects_function(r, f) = forall(x0: T, y0: T) {
            r(x0, y0) implies f(x0) = f(y0)
        }
        f(x) = f(y)
    }
}

/// Predicate invariance is function invariance for Boolean-valued functions.
theorem respects_predicate_eq_respects_function[T](r: (T, T) -> Bool, p: T -> Bool) {
    respects_predicate(r, p) = respects_function(r, p)
}

/// A relation-invariant predicate has equal truth values at any related pair.
theorem respects_predicate_step_eq[T](r: (T, T) -> Bool, p: T -> Bool, x: T, y: T) {
    respects_predicate(r, p) and r(x, y) implies p(x) = p(y)
} by {
    if respects_predicate(r, p) and r(x, y) {
        respects_function_step(r, p, x, y)
    }
}

/// A relation-invariant predicate transports truth forward.
theorem respects_predicate_forward[T](r: (T, T) -> Bool, p: T -> Bool, x: T, y: T) {
    respects_predicate(r, p) and r(x, y) and p(x) implies p(y)
} by {
    if respects_predicate(r, p) and r(x, y) and p(x) {
        respects_predicate_step_eq(r, p, x, y)
        p(x) = p(y)
        p(y)
    }
}

/// A relation-invariant predicate transports truth backward.
theorem respects_predicate_backward[T](r: (T, T) -> Bool, p: T -> Bool, x: T, y: T) {
    respects_predicate(r, p) and r(x, y) and p(y) implies p(x)
} by {
    if respects_predicate(r, p) and r(x, y) and p(y) {
        respects_predicate_step_eq(r, p, x, y)
        p(x) = p(y)
        p(x)
    }
}

/// Pulling back a relation-invariant predicate along a compatible function preserves invariance.
theorem predicate_pullback_respects_predicate_of_respects_equivalence[T, U](
    f: T -> U,
    r: (T, T) -> Bool,
    s: (U, U) -> Bool,
    p: U -> Bool
) {
    respects_equivalence(f, r, s) and respects_predicate(s, p) implies
    respects_predicate(r, predicate_pullback(f, p))
} by {
    if respects_equivalence(f, r, s) and respects_predicate(s, p) {
        forall(x: T, y: T) {
            if r(x, y) {
                respects_equivalence(f, r, s) = forall(a: T, b: T) {
                    r(a, b) implies s(f(a), f(b))
                }
                s(f(x), f(y))
                respects_predicate_step_eq(s, p, f(x), f(y))
                p(f(x)) = p(f(y))
                predicate_pullback(f, p, x) = p(f(x))
                predicate_pullback(f, p, y) = p(f(y))
                predicate_pullback(f, p, x) = predicate_pullback(f, p, y)
            }
        }
        respects_predicate(r, predicate_pullback(f, p)) =
            respects_function(r, predicate_pullback(f, p))
        respects_function(r, predicate_pullback(f, p)) = forall(x: T, y: T) {
            r(x, y) implies predicate_pullback(f, p, x) = predicate_pullback(f, p, y)
        }
        respects_predicate(r, predicate_pullback(f, p))
    }
}

/// Pulling back a relation-invariant predicate along any function preserves invariance for the pullback relation.
theorem predicate_pullback_respects_predicate[T, U](f: T -> U, s: (U, U) -> Bool, p: U -> Bool) {
    respects_predicate(s, p) implies respects_predicate(relation_pullback(f, s), predicate_pullback(f, p))
} by {
    if respects_predicate(s, p) {
        forall(x: T, y: T) {
            if relation_pullback(f, s, x, y) {
                relation_pullback(f, s, x, y) = s(f(x), f(y))
                s(f(x), f(y))
                respects_predicate_step_eq(s, p, f(x), f(y))
                p(f(x)) = p(f(y))
                predicate_pullback(f, p, x) = p(f(x))
                predicate_pullback(f, p, y) = p(f(y))
                predicate_pullback(f, p, x) = predicate_pullback(f, p, y)
            }
        }
        respects_predicate(relation_pullback(f, s), predicate_pullback(f, p)) =
            respects_function(relation_pullback(f, s), predicate_pullback(f, p))
        respects_function(relation_pullback(f, s), predicate_pullback(f, p)) = forall(x: T, y: T) {
            relation_pullback(f, s, x, y) implies
            predicate_pullback(f, p, x) = predicate_pullback(f, p, y)
        }
        respects_predicate(relation_pullback(f, s), predicate_pullback(f, p))
    }
}

/// A predicate invariant under a symmetric relation has invariant pushforward along an injective function.
theorem predicate_pushforward_respects_predicate_of_injective_symmetric[T, U](
    f: T -> U,
    r: (T, T) -> Bool,
    p: T -> Bool
) {
    is_injective_fn(f) and is_symmetric(r) and respects_predicate(r, p) implies
    respects_predicate(relation_pushforward(f, r), predicate_pushforward(f, p))
} by {
    if is_injective_fn(f) and is_symmetric(r) and respects_predicate(r, p) {
        forall(u: U, v: U) {
            if relation_pushforward(f, r, u, v) {
                relation_pushforward_has_preimages(f, r, u, v)
                let a: T satisfy {
                    exists(b: T) {
                        f(a) = u and f(b) = v and r(a, b)
                    }
                }
                let b: T satisfy {
                    f(a) = u and f(b) = v and r(a, b)
                }
                if predicate_pushforward(f, p, u) {
                    predicate_pushforward_has_preimage(f, p, u)
                    let x: T satisfy {
                        f(x) = u and p(x)
                    }
                    f(x) = f(a)
                    injective_fn_eq(f, x, a)
                    x = a
                    p(a)
                    respects_predicate_forward(r, p, a, b)
                    p(b)
                    predicate_pushforward_intro(f, p, b)
                    predicate_pushforward(f, p, f(b))
                    predicate_pushforward(f, p, v)
                }
                if predicate_pushforward(f, p, v) {
                    predicate_pushforward_has_preimage(f, p, v)
                    let y: T satisfy {
                        f(y) = v and p(y)
                    }
                    f(y) = f(b)
                    injective_fn_eq(f, y, b)
                    y = b
                    p(b)
                    symmetric_flip(r, a, b)
                    r(b, a)
                    respects_predicate_forward(r, p, b, a)
                    p(a)
                    predicate_pushforward_intro(f, p, a)
                    predicate_pushforward(f, p, f(a))
                    predicate_pushforward(f, p, u)
                }
                predicate_pushforward(f, p, u) = predicate_pushforward(f, p, v)
            }
        }
        respects_predicate(relation_pushforward(f, r), predicate_pushforward(f, p)) =
            respects_function(relation_pushforward(f, r), predicate_pushforward(f, p))
        respects_function(relation_pushforward(f, r), predicate_pushforward(f, p)) = forall(u: U, v: U) {
            relation_pushforward(f, r, u, v) implies
            predicate_pushforward(f, p, u) = predicate_pushforward(f, p, v)
        }
        respects_predicate(relation_pushforward(f, r), predicate_pushforward(f, p))
    }
}

/// A predicate invariant under an equivalence relation has invariant pushforward along an injective function.
theorem predicate_pushforward_respects_predicate_of_injective_equivalence[T, U](
    f: T -> U,
    r: (T, T) -> Bool,
    p: T -> Bool
) {
    is_injective_fn(f) and is_equivalence(r) and respects_predicate(r, p) implies
    respects_predicate(relation_pushforward(f, r), predicate_pushforward(f, p))
} by {
    if is_injective_fn(f) and is_equivalence(r) and respects_predicate(r, p) {
        equivalence_is_symmetric(r)
        is_symmetric(r)
        predicate_pushforward_respects_predicate_of_injective_symmetric(f, r, p)
        respects_predicate(relation_pushforward(f, r), predicate_pushforward(f, p))
    }
}

/// Pulling back along the identity function leaves a relation unchanged.
theorem relation_pullback_identity[T](r: (T, T) -> Bool) {
    relation_pullback(identity_fn[T], r) = r
} by {
    let u = relation_pullback(identity_fn[T], r)
    let v = r
    forall(x: T, y: T) {
        u(x, y) = r(identity_fn[T](x), identity_fn[T](y))
        identity_fn[T](x) = x
        identity_fn[T](y) = y
        u(x, y) = v(x, y)
    }
    binary_function_extensionality(u, v)
}

/// Pullback respects function composition.
theorem relation_pullback_compose[A, B, C](g: A -> B, f: B -> C, r: (C, C) -> Bool) {
    relation_pullback(compose(f, g), r) = relation_pullback(g, relation_pullback(f, r))
} by {
    let u = relation_pullback(compose(f, g), r)
    let v = relation_pullback(g, relation_pullback(f, r))
    forall(x: A, y: A) {
        u(x, y) = r(compose(f, g)(x), compose(f, g)(y))
        compose(f, g, x) = f(g(x))
        compose(f, g, y) = f(g(y))
        v(x, y) = relation_pullback(f, r, g(x), g(y))
        relation_pullback(f, r, g(x), g(y)) = r(f(g(x)), f(g(y)))
        u(x, y) = v(x, y)
    }
    binary_function_extensionality(u, v)
}

/// Pullback commutes with relation intersection.
theorem relation_pullback_intersection[A, B](f: A -> B, r: (B, B) -> Bool, s: (B, B) -> Bool) {
    relation_pullback(f, relation_intersection(r, s)) =
    relation_intersection(relation_pullback(f, r), relation_pullback(f, s))
} by {
    let u = relation_pullback(f, relation_intersection(r, s))
    let v = relation_intersection(relation_pullback(f, r), relation_pullback(f, s))
    forall(x: A, y: A) {
        if u(x, y) {
            relation_pullback(f, relation_intersection(r, s), x, y)
            relation_intersection(r, s, f(x), f(y))
            r(f(x), f(y))
            s(f(x), f(y))
            relation_pullback(f, r, x, y)
            relation_pullback(f, s, x, y)
            v(x, y)
        }
        if v(x, y) {
            relation_pullback(f, r, x, y)
            relation_pullback(f, s, x, y)
            r(f(x), f(y))
            s(f(x), f(y))
            relation_intersection(r, s, f(x), f(y))
            u(x, y)
        }
        u(x, y) = v(x, y)
    }
    binary_function_extensionality(u, v)
}

/// Pullback distributes over union.
theorem relation_pullback_union[A, B](f: A -> B, r: (B, B) -> Bool, s: (B, B) -> Bool) {
    relation_pullback(f, relation_union(r, s)) =
    relation_union(relation_pullback(f, r), relation_pullback(f, s))
} by {
    let u = relation_pullback(f, relation_union(r, s))
    let v = relation_union(relation_pullback(f, r), relation_pullback(f, s))
    forall(x: A, y: A) {
        u(x, y) = relation_union(r, s, f(x), f(y))
        relation_union(r, s, f(x), f(y)) = (r(f(x), f(y)) or s(f(x), f(y)))
        relation_pullback(f, r, x, y) = r(f(x), f(y))
        relation_pullback(f, s, x, y) = s(f(x), f(y))
        v(x, y) = (relation_pullback(f, r, x, y) or relation_pullback(f, s, x, y))
        u(x, y) = v(x, y)
    }
    binary_function_extensionality(u, v)
}

/// Pushforward commutes with converse.
theorem relation_pushforward_converse[A, B](f: A -> B, r: (A, A) -> Bool) {
    relation_pushforward(f, relation_converse(r)) = relation_converse(relation_pushforward(f, r))
} by {
    let u = relation_pushforward(f, relation_converse(r))
    let v = relation_converse(relation_pushforward(f, r))
    forall(a: B, b: B) {
        if u(a, b) {
            relation_pushforward_has_preimages(f, relation_converse(r), a, b)
            let x: A satisfy {
                exists(y0: A) {
                    f(x) = a and f(y0) = b and relation_converse(r, x, y0)
                }
            }
            let y: A satisfy {
                f(x) = a and f(y) = b and relation_converse(r, x, y)
            }
            r(y, x)
            relation_pushforward_intro(f, r, y, x)
            relation_pushforward(f, r, b, a)
            v(a, b)
        }
        if v(a, b) {
            relation_pushforward(f, r, b, a)
            relation_pushforward_has_preimages(f, r, b, a)
            let y: A satisfy {
                exists(x0: A) {
                    f(y) = b and f(x0) = a and r(y, x0)
                }
            }
            let x: A satisfy {
                f(y) = b and f(x) = a and r(y, x)
            }
            relation_converse(r, x, y)
            relation_pushforward_intro(f, relation_converse(r), x, y)
            relation_pushforward(f, relation_converse(r), a, b)
            u(a, b)
        }
        u(a, b) = v(a, b)
    }
    binary_function_extensionality(u, v)
}

/// Pushforward distributes over union.
theorem relation_pushforward_union[A, B](f: A -> B, r: (A, A) -> Bool, s: (A, A) -> Bool) {
    relation_pushforward(f, relation_union(r, s)) =
    relation_union(relation_pushforward(f, r), relation_pushforward(f, s))
} by {
    let u = relation_pushforward(f, relation_union(r, s))
    let v = relation_union(relation_pushforward(f, r), relation_pushforward(f, s))
    forall(a: B, b: B) {
        if u(a, b) {
            relation_pushforward_has_preimages(f, relation_union(r, s), a, b)
            let x: A satisfy {
                exists(y0: A) {
                    f(x) = a and f(y0) = b and relation_union(r, s, x, y0)
                }
            }
            let y: A satisfy {
                f(x) = a and f(y) = b and relation_union(r, s, x, y)
            }
            relation_union(r, s, x, y) = (r(x, y) or s(x, y))
            if r(x, y) {
                relation_pushforward_intro(f, r, x, y)
                relation_pushforward(f, r, a, b)
                v(a, b)
            }
            if s(x, y) {
                relation_pushforward_intro(f, s, x, y)
                relation_pushforward(f, s, a, b)
                v(a, b)
            }
            v(a, b)
        }
        if v(a, b) {
            v(a, b) = (relation_pushforward(f, r, a, b) or relation_pushforward(f, s, a, b))
            if relation_pushforward(f, r, a, b) {
                relation_pushforward_has_preimages(f, r, a, b)
                let x: A satisfy {
                    exists(y0: A) {
                        f(x) = a and f(y0) = b and r(x, y0)
                    }
                }
                let y: A satisfy {
                    f(x) = a and f(y) = b and r(x, y)
                }
                relation_union(r, s, x, y)
                relation_pushforward_intro(f, relation_union(r, s), x, y)
                u(a, b)
            }
            if relation_pushforward(f, s, a, b) {
                relation_pushforward_has_preimages(f, s, a, b)
                let x: A satisfy {
                    exists(y0: A) {
                        f(x) = a and f(y0) = b and s(x, y0)
                    }
                }
                let y: A satisfy {
                    f(x) = a and f(y) = b and s(x, y)
                }
                relation_union(r, s, x, y)
                relation_pushforward_intro(f, relation_union(r, s), x, y)
                u(a, b)
            }
            u(a, b)
        }
        u(a, b) = v(a, b)
    }
    binary_function_extensionality(u, v)
}

/// Pushforward of an intersection lies in the intersection of the pushforwards.
theorem relation_pushforward_intersection_subset[A, B](f: A -> B, r: (A, A) -> Bool, s: (A, A) -> Bool) {
    relation_subset(
        relation_pushforward(f, relation_intersection(r, s)),
        relation_intersection(relation_pushforward(f, r), relation_pushforward(f, s))
    )
} by {
    forall(a: B, b: B) {
        if relation_pushforward(f, relation_intersection(r, s), a, b) {
            relation_pushforward_has_preimages(f, relation_intersection(r, s), a, b)
            let x: A satisfy {
                exists(y0: A) {
                    f(x) = a and f(y0) = b and relation_intersection(r, s, x, y0)
                }
            }
            let y: A satisfy {
                f(x) = a and f(y) = b and relation_intersection(r, s, x, y)
            }
            relation_intersection(r, s, x, y) = (r(x, y) and s(x, y))
            relation_pushforward_intro(f, r, x, y)
            relation_pushforward(f, r, a, b)
            relation_pushforward_intro(f, s, x, y)
            relation_pushforward(f, s, a, b)
            relation_intersection(relation_pushforward(f, r), relation_pushforward(f, s), a, b)
        }
    }
}

/// Pullback is monotone with respect to relation inclusion.
theorem relation_pullback_monotone[A, B](f: A -> B, r: (B, B) -> Bool, s: (B, B) -> Bool) {
    relation_subset(r, s) implies relation_subset(relation_pullback(f, r), relation_pullback(f, s))
} by {
    if relation_subset(r, s) {
        forall(x: A, y: A) {
            if relation_pullback(f, r, x, y) {
                relation_subset_step(r, s, f(x), f(y))
                relation_pullback(f, s, x, y)
            }
        }
    }
}

/// Pullback preserves reflexive relations.
theorem relation_pullback_is_reflexive[A, B](f: A -> B, r: (B, B) -> Bool) {
    is_reflexive(r) implies is_reflexive(relation_pullback(f, r))
} by {
    if is_reflexive(r) {
        forall(x: A) {
            reflexive_self(r, f(x))
            relation_pullback(f, r, x, x)
        }
    }
}

/// Pullback preserves irreflexivity.
theorem relation_pullback_is_irreflexive[A, B](f: A -> B, r: (B, B) -> Bool) {
    is_irreflexive(r) implies is_irreflexive(relation_pullback(f, r))
} by {
    if is_irreflexive(r) {
        forall(x: A) {
            if relation_pullback(f, r, x, x) {
                relation_pullback(f, r, x, x) = r(f(x), f(x))
                r(f(x), f(x))
                false
            }
        }
    }
}

/// Pullback preserves symmetric relations.
theorem relation_pullback_is_symmetric[A, B](f: A -> B, r: (B, B) -> Bool) {
    is_symmetric(r) implies is_symmetric(relation_pullback(f, r))
} by {
    if is_symmetric(r) {
        forall(x: A, y: A) {
            if relation_pullback(f, r, x, y) {
                symmetric_flip(r, f(x), f(y))
                relation_pullback(f, r, y, x)
            }
        }
    }
}

/// Pullback preserves asymmetry.
theorem relation_pullback_is_asymmetric[A, B](f: A -> B, r: (B, B) -> Bool) {
    is_asymmetric(r) implies is_asymmetric(relation_pullback(f, r))
} by {
    if is_asymmetric(r) {
        is_asymmetric(r) = forall(x0: B, y0: B) {
            r(x0, y0) implies not r(y0, x0)
        }
        forall(x: A, y: A) {
            if relation_pullback(f, r, x, y) {
                relation_pullback(f, r, x, y) = r(f(x), f(y))
                r(f(x), f(y))
                not r(f(y), f(x))
                if relation_pullback(f, r, y, x) {
                    relation_pullback(f, r, y, x) = r(f(y), f(x))
                    r(f(y), f(x))
                    false
                }
            }
        }
    }
}

/// Pullback preserves transitive relations.
theorem relation_pullback_is_transitive[A, B](f: A -> B, r: (B, B) -> Bool) {
    is_transitive(r) implies is_transitive(relation_pullback(f, r))
} by {
    if is_transitive(r) {
        forall(x: A, y: A, z: A) {
            if relation_pullback(f, r, x, y) and relation_pullback(f, r, y, z) {
                r(f(x), f(y))
                r(f(y), f(z))
                transitive_step(r, f(x), f(y), f(z))
                r(f(x), f(z))
                relation_pullback(f, r, x, z)
            }
        }
    }
}

/// Pullback preserves totality.
theorem relation_pullback_is_total[A, B](f: A -> B, r: (B, B) -> Bool) {
    is_total(r) implies is_total(relation_pullback(f, r))
} by {
    if is_total(r) {
        is_total(r) = forall(x0: B, y0: B) {
            r(x0, y0) or r(y0, x0)
        }
        forall(x: A, y: A) {
            r(f(x), f(y)) or r(f(y), f(x))
            if r(f(x), f(y)) {
                relation_pullback(f, r, x, y)
                relation_pullback(f, r, x, y) or relation_pullback(f, r, y, x)
            } else {
                relation_pullback(f, r, y, x)
                relation_pullback(f, r, x, y) or relation_pullback(f, r, y, x)
            }
        }
    }
}

/// Pullback preserves equivalence relations.
theorem relation_pullback_is_equivalence[A, B](f: A -> B, r: (B, B) -> Bool) {
    is_equivalence(r) implies is_equivalence(relation_pullback(f, r))
} by {
    if is_equivalence(r) {
        equivalence_is_reflexive(r)
        relation_pullback_is_reflexive(f, r)
        is_reflexive(relation_pullback(f, r))
        equivalence_is_symmetric(r)
        relation_pullback_is_symmetric(f, r)
        is_symmetric(relation_pullback(f, r))
        equivalence_is_transitive(r)
        relation_pullback_is_transitive(f, r)
        is_transitive(relation_pullback(f, r))
        is_equivalence(relation_pullback(f, r))
    }
}

/// Pullback preserves partial equivalence relations.
theorem relation_pullback_is_partial_equivalence[A, B](f: A -> B, r: (B, B) -> Bool) {
    is_partial_equivalence(r) implies is_partial_equivalence(relation_pullback(f, r))
} by {
    if is_partial_equivalence(r) {
        partial_equivalence_is_symmetric(r)
        relation_pullback_is_symmetric(f, r)
        is_symmetric(relation_pullback(f, r))
        partial_equivalence_is_transitive(r)
        relation_pullback_is_transitive(f, r)
        is_transitive(relation_pullback(f, r))
        is_partial_equivalence(relation_pullback(f, r))
    }
}

/// Every function respects the pullback of a relation along itself.
theorem function_respects_pullback[T, U](f: T -> U, r: (U, U) -> Bool) {
    respects_equivalence(f, relation_pullback(f, r), r)
} by {
    relation_subset_refl(relation_pullback(f, r))
    respects_equivalence_eq_relation_subset_pullback(f, relation_pullback(f, r), r)
    respects_equivalence(f, relation_pullback(f, r), r)
}

/// The identity function respects any relation.
theorem identity_fn_respects_equivalence[T](r: (T, T) -> Bool) {
    respects_equivalence(identity_fn[T], r, r)
} by {
    relation_subset_refl(r)
    relation_pullback_identity(r)
    relation_subset(r, relation_pullback(identity_fn[T], r))
    respects_equivalence_eq_relation_subset_pullback(identity_fn[T], r, r)
    respects_equivalence(identity_fn[T], r, r)
}

/// Composition preserves the property of respecting equivalence relations.
theorem compose_respects_equivalence[A, B, C](
    f: B -> C,
    g: A -> B,
    r: (A, A) -> Bool,
    s: (B, B) -> Bool,
    t: (C, C) -> Bool
) {
    respects_equivalence(g, r, s) and respects_equivalence(f, s, t) implies
    respects_equivalence(compose(f, g), r, t)
} by {
    if respects_equivalence(g, r, s) and respects_equivalence(f, s, t) {
        respects_equivalence_eq_relation_subset_pullback(g, r, s)
        relation_subset(r, relation_pullback(g, s))
        respects_equivalence_eq_relation_subset_pullback(f, s, t)
        relation_subset(s, relation_pullback(f, t))
        relation_pullback_monotone(g, s, relation_pullback(f, t))
        relation_subset(relation_pullback(g, s), relation_pullback(g, relation_pullback(f, t)))
        relation_subset_trans(r, relation_pullback(g, s), relation_pullback(g, relation_pullback(f, t)))
        relation_subset(r, relation_pullback(g, relation_pullback(f, t)))
        relation_pullback_compose(g, f, t)
        relation_subset(r, relation_pullback(compose(f, g), t))
        respects_equivalence_eq_relation_subset_pullback(compose(f, g), r, t)
        respects_equivalence(compose(f, g), r, t)
    }
}

/// Injectivity identifies each equality pullback instance with ordinary equality.
theorem relation_pullback_eq_relation_of_injective_at[T, U](f: T -> U, x: T, y: T) {
    is_injective_fn(f) implies relation_pullback(f, eq_relation[U], x, y) = eq_relation[T](x, y)
} by {
    if is_injective_fn(f) {
        if relation_pullback(f, eq_relation[U], x, y) {
            relation_pullback(f, eq_relation[U], x, y) = eq_relation[U](f(x), f(y))
            eq_relation[U](f(x), f(y)) = (f(x) = f(y))
            f(x) = f(y)
            injective_fn_eq(f, x, y)
            x = y
            eq_relation[T](x, y)
        }
        if eq_relation[T](x, y) {
            eq_relation[T](x, y) = (x = y)
            x = y
            f(x) = f(y)
            eq_relation[U](f(x), f(y))
            relation_pullback(f, eq_relation[U], x, y)
        }
        relation_pullback(f, eq_relation[U], x, y) = eq_relation[T](x, y)
    }
}

/// Pulling back equality along an injective function recovers equality.
theorem relation_pullback_eq_relation_of_injective[T, U](f: T -> U) {
    is_injective_fn(f) implies relation_pullback(f, eq_relation[U]) = eq_relation[T]
} by {
    let u = relation_pullback(f, eq_relation[U])
    let v = eq_relation[T]
    if is_injective_fn(f) {
        forall(x: T, y: T) {
            relation_pullback_eq_relation_of_injective_at(f, x, y)
            u(x, y) = v(x, y)
        }
        binary_function_extensionality(u, v)
    }
}

/// Injective pullback preserves antisymmetry.
theorem relation_pullback_is_antisymmetric_of_injective[A, B](f: A -> B, r: (B, B) -> Bool) {
    is_injective_fn(f) and is_antisymmetric(r) implies is_antisymmetric(relation_pullback(f, r))
} by {
    if is_injective_fn(f) and is_antisymmetric(r) {
        forall(x: A, y: A) {
            if relation_pullback(f, r, x, y) and relation_pullback(f, r, y, x) {
                relation_pullback(f, r, x, y) = r(f(x), f(y))
                r(f(x), f(y))
                relation_pullback(f, r, y, x) = r(f(y), f(x))
                r(f(y), f(x))
                antisymmetric_eq(r, f(x), f(y))
                f(x) = f(y)
                injective_fn_eq(f, x, y)
                x = y
            }
        }
    }
}

/// Injective pullback preserves reflexive, transitive, and antisymmetric order data.
theorem relation_pullback_is_reflexive_transitive_antisymmetric_of_injective[A, B](f: A -> B, r: (B, B) -> Bool) {
    is_injective_fn(f) and is_reflexive(r) and is_transitive(r) and is_antisymmetric(r) implies
    is_reflexive(relation_pullback(f, r)) and
    is_transitive(relation_pullback(f, r)) and
    is_antisymmetric(relation_pullback(f, r))
} by {
    if is_injective_fn(f) and is_reflexive(r) and is_transitive(r) and is_antisymmetric(r) {
        relation_pullback_is_reflexive(f, r)
        is_reflexive(relation_pullback(f, r))
        relation_pullback_is_transitive(f, r)
        is_transitive(relation_pullback(f, r))
        relation_pullback_is_antisymmetric_of_injective(f, r)
        is_antisymmetric(relation_pullback(f, r))
    }
}

/// Injectivity identifies each reflexive-closure pullback instance with the pullback reflexive closure.
theorem relation_pullback_reflexive_closure_of_injective_at[A, B](f: A -> B, r: (B, B) -> Bool, x: A, y: A) {
    is_injective_fn(f) implies
    relation_pullback(f, relation_reflexive_closure(r), x, y) = relation_reflexive_closure(relation_pullback(f, r), x, y)
} by {
    if is_injective_fn(f) {
        relation_pullback_eq_relation_of_injective_at(f, x, y)
        relation_pullback(f, relation_reflexive_closure(r), x, y) = relation_reflexive_closure(r, f(x), f(y))
        relation_reflexive_closure(r, f(x), f(y)) = (eq_relation[B](f(x), f(y)) or r(f(x), f(y)))
        relation_pullback(f, eq_relation[B], x, y) = eq_relation[B](f(x), f(y))
        relation_pullback(f, eq_relation[B], x, y) = eq_relation[A](x, y)
        eq_relation[B](f(x), f(y)) = eq_relation[A](x, y)
        relation_pullback(f, r, x, y) = r(f(x), f(y))
        relation_pullback(f, relation_reflexive_closure(r), x, y) = (eq_relation[A](x, y) or relation_pullback(f, r, x, y))
        relation_reflexive_closure(relation_pullback(f, r), x, y) = (eq_relation[A](x, y) or relation_pullback(f, r, x, y))
        relation_pullback(f, relation_reflexive_closure(r), x, y) = relation_reflexive_closure(relation_pullback(f, r), x, y)
    }
}

/// Injective pullback commutes with reflexive closure.
theorem relation_pullback_reflexive_closure_of_injective[A, B](f: A -> B, r: (B, B) -> Bool) {
    is_injective_fn(f) implies relation_pullback(f, relation_reflexive_closure(r)) = relation_reflexive_closure(relation_pullback(f, r))
} by {
    let u = relation_pullback(f, relation_reflexive_closure(r))
    let v = relation_reflexive_closure(relation_pullback(f, r))
    if is_injective_fn(f) {
        forall(x: A, y: A) {
            relation_pullback_reflexive_closure_of_injective_at(f, r, x, y)
            u(x, y) = v(x, y)
        }
        binary_function_extensionality(u, v)
    }
}

/// Surjective pullback reflects reflexivity.
theorem relation_pullback_reflects_reflexive_of_surjective[A, B](f: A -> B, r: (B, B) -> Bool) {
    is_surjective_fn(f) and is_reflexive(relation_pullback(f, r)) implies is_reflexive(r)
} by {
    if is_surjective_fn(f) and is_reflexive(relation_pullback(f, r)) {
        forall(y: B) {
            surjective_fn_has_preimage(f, y)
            let x: A satisfy {
                f(x) = y
            }
            reflexive_self(relation_pullback(f, r), x)
            relation_pullback(f, r, x, x)
            relation_pullback(f, r, x, x) = r(f(x), f(x))
            f(x) = y
            r(y, y)
        }
    }
}

/// Surjective pullback reflects irreflexivity.
theorem relation_pullback_reflects_irreflexive_of_surjective[A, B](f: A -> B, r: (B, B) -> Bool) {
    is_surjective_fn(f) and is_irreflexive(relation_pullback(f, r)) implies is_irreflexive(r)
} by {
    if is_surjective_fn(f) and is_irreflexive(relation_pullback(f, r)) {
        forall(y: B) {
            surjective_fn_has_preimage(f, y)
            let x: A satisfy {
                f(x) = y
            }
            if r(y, y) {
                relation_pullback(f, r, x, x) = r(f(x), f(x))
                f(x) = y
                relation_pullback(f, r, x, x)
                false
            }
        }
    }
}

/// Surjective pullback preserves and reflects reflexivity.
theorem relation_pullback_is_reflexive_iff_of_surjective[A, B](f: A -> B, r: (B, B) -> Bool) {
    is_surjective_fn(f) implies is_reflexive(relation_pullback(f, r)) = is_reflexive(r)
} by {
    if is_surjective_fn(f) {
        if is_reflexive(relation_pullback(f, r)) {
            relation_pullback_reflects_reflexive_of_surjective(f, r)
            is_reflexive(r)
        }
        if is_reflexive(r) {
            relation_pullback_is_reflexive(f, r)
            is_reflexive(relation_pullback(f, r))
        }
        is_reflexive(relation_pullback(f, r)) = is_reflexive(r)
    }
}

/// Surjective pullback preserves and reflects irreflexivity.
theorem relation_pullback_is_irreflexive_iff_of_surjective[A, B](f: A -> B, r: (B, B) -> Bool) {
    is_surjective_fn(f) implies is_irreflexive(relation_pullback(f, r)) = is_irreflexive(r)
} by {
    if is_surjective_fn(f) {
        if is_irreflexive(relation_pullback(f, r)) {
            relation_pullback_reflects_irreflexive_of_surjective(f, r)
            is_irreflexive(r)
        }
        if is_irreflexive(r) {
            relation_pullback_is_irreflexive(f, r)
            is_irreflexive(relation_pullback(f, r))
        }
        is_irreflexive(relation_pullback(f, r)) = is_irreflexive(r)
    }
}

/// Surjective pullback reflects symmetry.
theorem relation_pullback_reflects_symmetric_of_surjective[A, B](f: A -> B, r: (B, B) -> Bool) {
    is_surjective_fn(f) and is_symmetric(relation_pullback(f, r)) implies is_symmetric(r)
} by {
    if is_surjective_fn(f) and is_symmetric(relation_pullback(f, r)) {
        forall(y1: B, y2: B) {
            if r(y1, y2) {
                surjective_fn_has_preimage(f, y1)
                let x1: A satisfy {
                    f(x1) = y1
                }
                surjective_fn_has_preimage(f, y2)
                let x2: A satisfy {
                    f(x2) = y2
                }
                relation_pullback(f, r, x1, x2) = r(f(x1), f(x2))
                f(x1) = y1
                f(x2) = y2
                relation_pullback(f, r, x1, x2)
                symmetric_flip(relation_pullback(f, r), x1, x2)
                relation_pullback(f, r, x2, x1)
                relation_pullback(f, r, x2, x1) = r(f(x2), f(x1))
                r(y2, y1)
            }
        }
    }
}

/// Surjective pullback reflects asymmetry.
theorem relation_pullback_reflects_asymmetric_of_surjective[A, B](f: A -> B, r: (B, B) -> Bool) {
    is_surjective_fn(f) and is_asymmetric(relation_pullback(f, r)) implies is_asymmetric(r)
} by {
    if is_surjective_fn(f) and is_asymmetric(relation_pullback(f, r)) {
        is_asymmetric(relation_pullback(f, r)) = forall(x0: A, y0: A) {
            relation_pullback(f, r, x0, y0) implies not relation_pullback(f, r, y0, x0)
        }
        forall(y1: B, y2: B) {
            if r(y1, y2) {
                surjective_fn_has_preimage(f, y1)
                let x1: A satisfy {
                    f(x1) = y1
                }
                surjective_fn_has_preimage(f, y2)
                let x2: A satisfy {
                    f(x2) = y2
                }
                relation_pullback(f, r, x1, x2) = r(f(x1), f(x2))
                f(x1) = y1
                f(x2) = y2
                relation_pullback(f, r, x1, x2)
                if r(y2, y1) {
                    relation_pullback(f, r, x2, x1) = r(f(x2), f(x1))
                    relation_pullback(f, r, x2, x1)
                    false
                }
            }
        }
    }
}

/// Surjective pullback preserves and reflects symmetry.
theorem relation_pullback_is_symmetric_iff_of_surjective[A, B](f: A -> B, r: (B, B) -> Bool) {
    is_surjective_fn(f) implies is_symmetric(relation_pullback(f, r)) = is_symmetric(r)
} by {
    if is_surjective_fn(f) {
        if is_symmetric(relation_pullback(f, r)) {
            relation_pullback_reflects_symmetric_of_surjective(f, r)
            is_symmetric(r)
        }
        if is_symmetric(r) {
            relation_pullback_is_symmetric(f, r)
            is_symmetric(relation_pullback(f, r))
        }
        is_symmetric(relation_pullback(f, r)) = is_symmetric(r)
    }
}

/// Surjective pullback preserves and reflects asymmetry.
theorem relation_pullback_is_asymmetric_iff_of_surjective[A, B](f: A -> B, r: (B, B) -> Bool) {
    is_surjective_fn(f) implies is_asymmetric(relation_pullback(f, r)) = is_asymmetric(r)
} by {
    if is_surjective_fn(f) {
        if is_asymmetric(relation_pullback(f, r)) {
            relation_pullback_reflects_asymmetric_of_surjective(f, r)
            is_asymmetric(r)
        }
        if is_asymmetric(r) {
            relation_pullback_is_asymmetric(f, r)
            is_asymmetric(relation_pullback(f, r))
        }
        is_asymmetric(relation_pullback(f, r)) = is_asymmetric(r)
    }
}

/// Surjective pullback reflects transitivity.
theorem relation_pullback_reflects_transitive_of_surjective[A, B](f: A -> B, r: (B, B) -> Bool) {
    is_surjective_fn(f) and is_transitive(relation_pullback(f, r)) implies is_transitive(r)
} by {
    if is_surjective_fn(f) and is_transitive(relation_pullback(f, r)) {
        forall(y1: B, y2: B, y3: B) {
            if r(y1, y2) and r(y2, y3) {
                surjective_fn_has_preimage(f, y1)
                let x1: A satisfy {
                    f(x1) = y1
                }
                surjective_fn_has_preimage(f, y2)
                let x2: A satisfy {
                    f(x2) = y2
                }
                surjective_fn_has_preimage(f, y3)
                let x3: A satisfy {
                    f(x3) = y3
                }
                relation_pullback(f, r, x1, x2) = r(f(x1), f(x2))
                f(x1) = y1
                f(x2) = y2
                relation_pullback(f, r, x1, x2)
                relation_pullback(f, r, x2, x3) = r(f(x2), f(x3))
                f(x3) = y3
                relation_pullback(f, r, x2, x3)
                transitive_step(relation_pullback(f, r), x1, x2, x3)
                relation_pullback(f, r, x1, x3)
                relation_pullback(f, r, x1, x3) = r(f(x1), f(x3))
                r(y1, y3)
            }
        }
    }
}

/// Surjective pullback reflects totality.
theorem relation_pullback_reflects_total_of_surjective[A, B](f: A -> B, r: (B, B) -> Bool) {
    is_surjective_fn(f) and is_total(relation_pullback(f, r)) implies is_total(r)
} by {
    if is_surjective_fn(f) and is_total(relation_pullback(f, r)) {
        is_total(relation_pullback(f, r)) = forall(x0: A, y0: A) {
            relation_pullback(f, r, x0, y0) or relation_pullback(f, r, y0, x0)
        }
        forall(y1: B, y2: B) {
            surjective_fn_has_preimage(f, y1)
            let x1: A satisfy {
                f(x1) = y1
            }
            surjective_fn_has_preimage(f, y2)
            let x2: A satisfy {
                f(x2) = y2
            }
            relation_pullback(f, r, x1, x2) or relation_pullback(f, r, x2, x1)
            if relation_pullback(f, r, x1, x2) {
                relation_pullback(f, r, x1, x2) = r(f(x1), f(x2))
                f(x1) = y1
                f(x2) = y2
                r(y1, y2)
                r(y1, y2) or r(y2, y1)
            } else {
                relation_pullback(f, r, x2, x1) = r(f(x2), f(x1))
                r(y2, y1)
                r(y1, y2) or r(y2, y1)
            }
        }
    }
}

/// Surjective pullback preserves and reflects transitivity.
theorem relation_pullback_is_transitive_iff_of_surjective[A, B](f: A -> B, r: (B, B) -> Bool) {
    is_surjective_fn(f) implies is_transitive(relation_pullback(f, r)) = is_transitive(r)
} by {
    if is_surjective_fn(f) {
        if is_transitive(relation_pullback(f, r)) {
            relation_pullback_reflects_transitive_of_surjective(f, r)
            is_transitive(r)
        }
        if is_transitive(r) {
            relation_pullback_is_transitive(f, r)
            is_transitive(relation_pullback(f, r))
        }
        is_transitive(relation_pullback(f, r)) = is_transitive(r)
    }
}

/// Surjective pullback preserves and reflects totality.
theorem relation_pullback_is_total_iff_of_surjective[A, B](f: A -> B, r: (B, B) -> Bool) {
    is_surjective_fn(f) implies is_total(relation_pullback(f, r)) = is_total(r)
} by {
    if is_surjective_fn(f) {
        if is_total(relation_pullback(f, r)) {
            relation_pullback_reflects_total_of_surjective(f, r)
            is_total(r)
        }
        if is_total(r) {
            relation_pullback_is_total(f, r)
            is_total(relation_pullback(f, r))
        }
        is_total(relation_pullback(f, r)) = is_total(r)
    }
}

/// Surjective pullback reflects antisymmetry.
theorem relation_pullback_reflects_antisymmetric_of_surjective[A, B](f: A -> B, r: (B, B) -> Bool) {
    is_surjective_fn(f) and is_antisymmetric(relation_pullback(f, r)) implies is_antisymmetric(r)
} by {
    if is_surjective_fn(f) and is_antisymmetric(relation_pullback(f, r)) {
        forall(y1: B, y2: B) {
            if r(y1, y2) and r(y2, y1) {
                surjective_fn_has_preimage(f, y1)
                let x1: A satisfy {
                    f(x1) = y1
                }
                surjective_fn_has_preimage(f, y2)
                let x2: A satisfy {
                    f(x2) = y2
                }
                relation_pullback(f, r, x1, x2) = r(f(x1), f(x2))
                f(x1) = y1
                f(x2) = y2
                relation_pullback(f, r, x1, x2)
                relation_pullback(f, r, x2, x1) = r(f(x2), f(x1))
                relation_pullback(f, r, x2, x1)
                antisymmetric_eq(relation_pullback(f, r), x1, x2)
                x1 = x2
                f(x1) = f(x2)
                y1 = y2
            }
        }
    }
}

/// Surjective pullback reflects equivalence relations.
theorem relation_pullback_reflects_equivalence_of_surjective[A, B](f: A -> B, r: (B, B) -> Bool) {
    is_surjective_fn(f) and is_equivalence(relation_pullback(f, r)) implies is_equivalence(r)
} by {
    if is_surjective_fn(f) and is_equivalence(relation_pullback(f, r)) {
        equivalence_is_reflexive(relation_pullback(f, r))
        relation_pullback_reflects_reflexive_of_surjective(f, r)
        is_reflexive(r)
        equivalence_is_symmetric(relation_pullback(f, r))
        relation_pullback_reflects_symmetric_of_surjective(f, r)
        is_symmetric(r)
        equivalence_is_transitive(relation_pullback(f, r))
        relation_pullback_reflects_transitive_of_surjective(f, r)
        is_transitive(r)
        is_equivalence(r)
    }
}

/// Surjective pullback preserves and reflects equivalence relations.
theorem relation_pullback_is_equivalence_iff_of_surjective[A, B](f: A -> B, r: (B, B) -> Bool) {
    is_surjective_fn(f) implies is_equivalence(relation_pullback(f, r)) = is_equivalence(r)
} by {
    if is_surjective_fn(f) {
        if is_equivalence(relation_pullback(f, r)) {
            relation_pullback_reflects_equivalence_of_surjective(f, r)
            is_equivalence(r)
        }
        if is_equivalence(r) {
            relation_pullback_is_equivalence(f, r)
            is_equivalence(relation_pullback(f, r))
        }
        is_equivalence(relation_pullback(f, r)) = is_equivalence(r)
    }
}

/// Surjective pullback reflects partial equivalence relations.
theorem relation_pullback_reflects_partial_equivalence_of_surjective[A, B](f: A -> B, r: (B, B) -> Bool) {
    is_surjective_fn(f) and is_partial_equivalence(relation_pullback(f, r)) implies is_partial_equivalence(r)
} by {
    if is_surjective_fn(f) and is_partial_equivalence(relation_pullback(f, r)) {
        partial_equivalence_is_symmetric(relation_pullback(f, r))
        relation_pullback_reflects_symmetric_of_surjective(f, r)
        is_symmetric(r)
        partial_equivalence_is_transitive(relation_pullback(f, r))
        relation_pullback_reflects_transitive_of_surjective(f, r)
        is_transitive(r)
        is_partial_equivalence(r)
    }
}

/// Surjective pullback preserves and reflects partial equivalence relations.
theorem relation_pullback_is_partial_equivalence_iff_of_surjective[A, B](f: A -> B, r: (B, B) -> Bool) {
    is_surjective_fn(f) implies
    is_partial_equivalence(relation_pullback(f, r)) = is_partial_equivalence(r)
} by {
    if is_surjective_fn(f) {
        if is_partial_equivalence(relation_pullback(f, r)) {
            relation_pullback_reflects_partial_equivalence_of_surjective(f, r)
            is_partial_equivalence(r)
        }
        if is_partial_equivalence(r) {
            relation_pullback_is_partial_equivalence(f, r)
            is_partial_equivalence(relation_pullback(f, r))
        }
        is_partial_equivalence(relation_pullback(f, r)) = is_partial_equivalence(r)
    }
}

/// Bijective pullback preserves and reflects antisymmetry.
theorem relation_pullback_is_antisymmetric_iff_of_bijection[A, B](f: A -> B, r: (B, B) -> Bool) {
    is_bijection_fn(f) implies is_antisymmetric(relation_pullback(f, r)) = is_antisymmetric(r)
} by {
    if is_bijection_fn(f) {
        if is_antisymmetric(relation_pullback(f, r)) {
            bijection_fn_is_surjective(f)
            relation_pullback_reflects_antisymmetric_of_surjective(f, r)
            is_antisymmetric(r)
        }
        if is_antisymmetric(r) {
            bijection_fn_is_injective(f)
            relation_pullback_is_antisymmetric_of_injective(f, r)
            is_antisymmetric(relation_pullback(f, r))
        }
        is_antisymmetric(relation_pullback(f, r)) = is_antisymmetric(r)
    }
}

/// Bijective pullback preserves and reflects reflexive, transitive, and antisymmetric order data.
theorem relation_pullback_is_reflexive_transitive_antisymmetric_iff_of_bijection[A, B](f: A -> B, r: (B, B) -> Bool) {
    is_bijection_fn(f) implies
    (is_reflexive(relation_pullback(f, r)) and
     is_transitive(relation_pullback(f, r)) and
     is_antisymmetric(relation_pullback(f, r))) =
    (is_reflexive(r) and is_transitive(r) and is_antisymmetric(r))
} by {
    if is_bijection_fn(f) {
        bijection_fn_is_surjective(f)
        relation_pullback_is_reflexive_iff_of_surjective(f, r)
        is_reflexive(relation_pullback(f, r)) = is_reflexive(r)
        relation_pullback_is_transitive_iff_of_surjective(f, r)
        is_transitive(relation_pullback(f, r)) = is_transitive(r)
        relation_pullback_is_antisymmetric_iff_of_bijection(f, r)
        is_antisymmetric(relation_pullback(f, r)) = is_antisymmetric(r)
        (is_reflexive(relation_pullback(f, r)) and
         is_transitive(relation_pullback(f, r)) and
         is_antisymmetric(relation_pullback(f, r))) =
        (is_reflexive(r) and is_transitive(r) and is_antisymmetric(r))
    }
}

/// True if a unary operation sends related inputs to related outputs.
define respects_unary_op[T](r: (T, T) -> Bool, op: T -> T) -> Bool {
    forall(x: T, y: T) {
        r(x, y) implies r(op(x), op(y))
    }
}

/// True if a binary operation sends pairwise related inputs to related outputs.
define respects_binary_op[T](r: (T, T) -> Bool, op: (T, T) -> T) -> Bool {
    forall(x1: T, y1: T, x2: T, y2: T) {
        r(x1, y1) and r(x2, y2) implies r(op(x1, x2), op(y1, y2))
    }
}

/// True if a function intertwines two unary operations.
define preserves_unary_op[A, B](f: A -> B, op_a: A -> A, op_b: B -> B) -> Bool {
    forall(x: A) {
        f(op_a(x)) = op_b(f(x))
    }
}

/// True if a function intertwines two binary operations.
define preserves_binary_op[A, B](f: A -> B, op_a: (A, A) -> A, op_b: (B, B) -> B) -> Bool {
    forall(x: A, y: A) {
        f(op_a(x, y)) = op_b(f(x), f(y))
    }
}

/// True if a relation is an equivalence relation compatible with a unary operation.
define is_unary_congruence[T](r: (T, T) -> Bool, op: T -> T) -> Bool {
    is_equivalence(r) and respects_unary_op(r, op)
}

/// True if a relation is an equivalence relation compatible with a binary operation.
define is_binary_congruence[T](r: (T, T) -> Bool, op: (T, T) -> T) -> Bool {
    is_equivalence(r) and respects_binary_op(r, op)
}

/// Equality of functions transports equivalence-respect.
theorem respects_equivalence_eq_function[T, U](
    f: T -> U,
    g: T -> U,
    r: (T, T) -> Bool,
    s: (U, U) -> Bool
) {
    f = g and respects_equivalence(f, r, s) implies respects_equivalence(g, r, s)
} by {
    if f = g and respects_equivalence(f, r, s) {
        respects_equivalence(g, r, s)
    }
}

/// Equality of source relations transports equivalence-respect.
theorem respects_equivalence_eq_source[T, U](
    f: T -> U,
    r: (T, T) -> Bool,
    q: (T, T) -> Bool,
    s: (U, U) -> Bool
) {
    r = q and respects_equivalence(f, r, s) implies respects_equivalence(f, q, s)
} by {
    if r = q and respects_equivalence(f, r, s) {
        respects_equivalence(f, q, s)
    }
}

/// Equality of target relations transports equivalence-respect.
theorem respects_equivalence_eq_target[T, U](
    f: T -> U,
    r: (T, T) -> Bool,
    s: (U, U) -> Bool,
    q: (U, U) -> Bool
) {
    s = q and respects_equivalence(f, r, s) implies respects_equivalence(f, r, q)
} by {
    if s = q and respects_equivalence(f, r, s) {
        respects_equivalence(f, r, q)
    }
}

/// Equality of all data transports equivalence-respect.
theorem respects_equivalence_eq[T, U](
    f: T -> U,
    g: T -> U,
    r: (T, T) -> Bool,
    q: (T, T) -> Bool,
    s: (U, U) -> Bool,
    t: (U, U) -> Bool
) {
    f = g and r = q and s = t and respects_equivalence(f, r, s) implies respects_equivalence(g, q, t)
} by {
    if f = g and r = q and s = t and respects_equivalence(f, r, s) {
        respects_equivalence(g, q, t)
    }
}

/// Equality of relations transports function-respect.
theorem respects_function_eq_relation[T, U](r: (T, T) -> Bool, q: (T, T) -> Bool, f: T -> U) {
    r = q and respects_function(r, f) implies respects_function(q, f)
} by {
    if r = q and respects_function(r, f) {
        respects_function(q, f)
    }
}

/// Equality of functions transports function-respect.
theorem respects_function_eq_function[T, U](r: (T, T) -> Bool, f: T -> U, g: T -> U) {
    f = g and respects_function(r, f) implies respects_function(r, g)
} by {
    if f = g and respects_function(r, f) {
        respects_function(r, g)
    }
}

/// Equality of relations transports predicate-respect.
theorem respects_predicate_eq_relation[T](r: (T, T) -> Bool, q: (T, T) -> Bool, p: T -> Bool) {
    r = q and respects_predicate(r, p) implies respects_predicate(q, p)
} by {
    if r = q and respects_predicate(r, p) {
        respects_predicate(q, p)
    }
}

/// Equality of predicates transports predicate-respect.
theorem respects_predicate_eq_predicate[T](r: (T, T) -> Bool, p: T -> Bool, q: T -> Bool) {
    p = q and respects_predicate(r, p) implies respects_predicate(r, q)
} by {
    if p = q and respects_predicate(r, p) {
        respects_predicate(r, q)
    }
}

/// Equality of functions transports unary-operation preservation.
theorem preserves_unary_op_eq_function[A, B](f: A -> B, g: A -> B, op_a: A -> A, op_b: B -> B) {
    f = g and preserves_unary_op(f, op_a, op_b) implies preserves_unary_op(g, op_a, op_b)
} by {
    if f = g and preserves_unary_op(f, op_a, op_b) {
        preserves_unary_op(g, op_a, op_b)
    }
}

/// Equality of domain operations transports unary-operation preservation.
theorem preserves_unary_op_eq_domain_op[A, B](f: A -> B, op_a: A -> A, op_c: A -> A, op_b: B -> B) {
    op_a = op_c and preserves_unary_op(f, op_a, op_b) implies preserves_unary_op(f, op_c, op_b)
} by {
    if op_a = op_c and preserves_unary_op(f, op_a, op_b) {
        preserves_unary_op(f, op_c, op_b)
    }
}

/// Equality of codomain operations transports unary-operation preservation.
theorem preserves_unary_op_eq_codomain_op[A, B](f: A -> B, op_a: A -> A, op_b: B -> B, op_c: B -> B) {
    op_b = op_c and preserves_unary_op(f, op_a, op_b) implies preserves_unary_op(f, op_a, op_c)
} by {
    if op_b = op_c and preserves_unary_op(f, op_a, op_b) {
        preserves_unary_op(f, op_a, op_c)
    }
}

/// Equality of functions transports binary-operation preservation.
theorem preserves_binary_op_eq_function[A, B](
    f: A -> B,
    g: A -> B,
    op_a: (A, A) -> A,
    op_b: (B, B) -> B
) {
    f = g and preserves_binary_op(f, op_a, op_b) implies preserves_binary_op(g, op_a, op_b)
} by {
    if f = g and preserves_binary_op(f, op_a, op_b) {
        preserves_binary_op(g, op_a, op_b)
    }
}

/// Equality of domain operations transports binary-operation preservation.
theorem preserves_binary_op_eq_domain_op[A, B](
    f: A -> B,
    op_a: (A, A) -> A,
    op_c: (A, A) -> A,
    op_b: (B, B) -> B
) {
    op_a = op_c and preserves_binary_op(f, op_a, op_b) implies preserves_binary_op(f, op_c, op_b)
} by {
    if op_a = op_c and preserves_binary_op(f, op_a, op_b) {
        preserves_binary_op(f, op_c, op_b)
    }
}

/// Equality of codomain operations transports binary-operation preservation.
theorem preserves_binary_op_eq_codomain_op[A, B](
    f: A -> B,
    op_a: (A, A) -> A,
    op_b: (B, B) -> B,
    op_c: (B, B) -> B
) {
    op_b = op_c and preserves_binary_op(f, op_a, op_b) implies preserves_binary_op(f, op_a, op_c)
} by {
    if op_b = op_c and preserves_binary_op(f, op_a, op_b) {
        preserves_binary_op(f, op_a, op_c)
    }
}

/// Equality of relations transports unary-operation compatibility.
theorem respects_unary_op_eq_relation[T](r: (T, T) -> Bool, q: (T, T) -> Bool, op: T -> T) {
    r = q and respects_unary_op(r, op) implies respects_unary_op(q, op)
} by {
    if r = q and respects_unary_op(r, op) {
        respects_unary_op(q, op)
    }
}

/// Equality of unary operations transports unary-operation compatibility.
theorem respects_unary_op_eq_op[T](r: (T, T) -> Bool, op: T -> T, op2: T -> T) {
    op = op2 and respects_unary_op(r, op) implies respects_unary_op(r, op2)
} by {
    if op = op2 and respects_unary_op(r, op) {
        respects_unary_op(r, op2)
    }
}

/// Equality of relations and operations transports unary-operation compatibility.
theorem respects_unary_op_eq[T](r: (T, T) -> Bool, q: (T, T) -> Bool, op: T -> T, op2: T -> T) {
    r = q and op = op2 and respects_unary_op(r, op) implies respects_unary_op(q, op2)
} by {
    if r = q and op = op2 and respects_unary_op(r, op) {
        respects_unary_op(q, op2)
    }
}

/// Equality of relations transports binary-operation compatibility.
theorem respects_binary_op_eq_relation[T](r: (T, T) -> Bool, q: (T, T) -> Bool, op: (T, T) -> T) {
    r = q and respects_binary_op(r, op) implies respects_binary_op(q, op)
} by {
    if r = q and respects_binary_op(r, op) {
        respects_binary_op(q, op)
    }
}

/// Equality of binary operations transports binary-operation compatibility.
theorem respects_binary_op_eq_op[T](r: (T, T) -> Bool, op: (T, T) -> T, op2: (T, T) -> T) {
    op = op2 and respects_binary_op(r, op) implies respects_binary_op(r, op2)
} by {
    if op = op2 and respects_binary_op(r, op) {
        respects_binary_op(r, op2)
    }
}

/// Equality of relations and operations transports binary-operation compatibility.
theorem respects_binary_op_eq[T](r: (T, T) -> Bool, q: (T, T) -> Bool, op: (T, T) -> T, op2: (T, T) -> T) {
    r = q and op = op2 and respects_binary_op(r, op) implies respects_binary_op(q, op2)
} by {
    if r = q and op = op2 and respects_binary_op(r, op) {
        respects_binary_op(q, op2)
    }
}

/// Equality of relations transports unary congruence.
theorem unary_congruence_eq_relation[T](r: (T, T) -> Bool, q: (T, T) -> Bool, op: T -> T) {
    r = q and is_unary_congruence(r, op) implies is_unary_congruence(q, op)
} by {
    if r = q and is_unary_congruence(r, op) {
        is_unary_congruence(q, op)
    }
}

/// Equality of unary operations transports unary congruence.
theorem unary_congruence_eq_op[T](r: (T, T) -> Bool, op: T -> T, op2: T -> T) {
    op = op2 and is_unary_congruence(r, op) implies is_unary_congruence(r, op2)
} by {
    if op = op2 and is_unary_congruence(r, op) {
        is_unary_congruence(r, op2)
    }
}

/// Equality of relations and operations transports unary congruence.
theorem unary_congruence_eq[T](r: (T, T) -> Bool, q: (T, T) -> Bool, op: T -> T, op2: T -> T) {
    r = q and op = op2 and is_unary_congruence(r, op) implies is_unary_congruence(q, op2)
} by {
    if r = q and op = op2 and is_unary_congruence(r, op) {
        is_unary_congruence(q, op2)
    }
}

/// Equality of relations transports binary congruence.
theorem binary_congruence_eq_relation[T](r: (T, T) -> Bool, q: (T, T) -> Bool, op: (T, T) -> T) {
    r = q and is_binary_congruence(r, op) implies is_binary_congruence(q, op)
} by {
    if r = q and is_binary_congruence(r, op) {
        is_binary_congruence(q, op)
    }
}

/// Equality of binary operations transports binary congruence.
theorem binary_congruence_eq_op[T](r: (T, T) -> Bool, op: (T, T) -> T, op2: (T, T) -> T) {
    op = op2 and is_binary_congruence(r, op) implies is_binary_congruence(r, op2)
} by {
    if op = op2 and is_binary_congruence(r, op) {
        is_binary_congruence(r, op2)
    }
}

/// Equality of relations and operations transports binary congruence.
theorem binary_congruence_eq[T](r: (T, T) -> Bool, q: (T, T) -> Bool, op: (T, T) -> T, op2: (T, T) -> T) {
    r = q and op = op2 and is_binary_congruence(r, op) implies is_binary_congruence(q, op2)
} by {
    if r = q and op = op2 and is_binary_congruence(r, op) {
        is_binary_congruence(q, op2)
    }
}

/// A unary-compatible relation may be applied at any related pair.
theorem respects_unary_op_step[T](r: (T, T) -> Bool, op: T -> T, x: T, y: T) {
    respects_unary_op(r, op) and r(x, y) implies r(op(x), op(y))
} by {
    if respects_unary_op(r, op) and r(x, y) {
        respects_unary_op(r, op) = forall(x0: T, y0: T) {
            r(x0, y0) implies r(op(x0), op(y0))
        }
        r(op(x), op(y))
    }
}

/// A binary-compatible relation may be applied at any pair of related inputs.
theorem respects_binary_op_step[T](r: (T, T) -> Bool, op: (T, T) -> T, x1: T, y1: T, x2: T, y2: T) {
    respects_binary_op(r, op) and r(x1, y1) and r(x2, y2) implies r(op(x1, x2), op(y1, y2))
} by {
    if respects_binary_op(r, op) and r(x1, y1) and r(x2, y2) {
        respects_binary_op(r, op) = forall(a1: T, b1: T, a2: T, b2: T) {
            r(a1, b1) and r(a2, b2) implies r(op(a1, a2), op(b1, b2))
        }
        r(op(x1, x2), op(y1, y2))
    }
}

/// An operation-preserving map may be rewritten at any input.
theorem preserves_unary_op_step[A, B](f: A -> B, op_a: A -> A, op_b: B -> B, x: A) {
    preserves_unary_op(f, op_a, op_b) implies f(op_a(x)) = op_b(f(x))
} by {
    if preserves_unary_op(f, op_a, op_b) {
        preserves_unary_op(f, op_a, op_b) = forall(x0: A) {
            f(op_a(x0)) = op_b(f(x0))
        }
        f(op_a(x)) = op_b(f(x))
    }
}

/// A binary-operation-preserving map may be rewritten at any pair of inputs.
theorem preserves_binary_op_step[A, B](f: A -> B, op_a: (A, A) -> A, op_b: (B, B) -> B, x: A, y: A) {
    preserves_binary_op(f, op_a, op_b) implies f(op_a(x, y)) = op_b(f(x), f(y))
} by {
    if preserves_binary_op(f, op_a, op_b) {
        preserves_binary_op(f, op_a, op_b) = forall(x0: A, y0: A) {
            f(op_a(x0, y0)) = op_b(f(x0), f(y0))
        }
        f(op_a(x, y)) = op_b(f(x), f(y))
    }
}

/// Every relation is compatible with the identity unary operation.
theorem identity_unary_op_respects_relation[T](r: (T, T) -> Bool) {
    respects_unary_op(r, identity_fn[T])
} by {
    forall(x: T, y: T) {
        if r(x, y) {
            identity_fn[T](x) = x
            identity_fn[T](y) = y
            r(identity_fn[T](x), identity_fn[T](y))
        }
    }
}

/// Compatibility with unary operations is closed under composition.
theorem respects_unary_op_compose[T](r: (T, T) -> Bool, op1: T -> T, op2: T -> T) {
    respects_unary_op(r, op1) and respects_unary_op(r, op2) implies
    respects_unary_op(r, compose(op1, op2))
} by {
    if respects_unary_op(r, op1) and respects_unary_op(r, op2) {
        forall(x: T, y: T) {
            if r(x, y) {
                respects_unary_op_step(r, op2, x, y)
                r(op2(x), op2(y))
                respects_unary_op_step(r, op1, op2(x), op2(y))
                r(op1(op2(x)), op1(op2(y)))
                compose(op1, op2, x) = op1(op2(x))
                compose(op1, op2, y) = op1(op2(y))
                r(compose(op1, op2)(x), compose(op1, op2)(y))
            }
        }
    }
}

/// The identity function preserves any unary operation.
theorem identity_fn_preserves_unary_op[T](op: T -> T) {
    preserves_unary_op(identity_fn[T], op, op)
} by {
    forall(x: T) {
        identity_fn[T](op(x)) = op(x)
        identity_fn[T](x) = x
        op(identity_fn[T](x)) = op(x)
        identity_fn[T](op(x)) = op(identity_fn[T](x))
    }
}

/// The identity function preserves any binary operation.
theorem identity_fn_preserves_binary_op[T](op: (T, T) -> T) {
    preserves_binary_op(identity_fn[T], op, op)
} by {
    forall(x: T, y: T) {
        identity_fn[T](op(x, y)) = op(x, y)
        identity_fn[T](x) = x
        identity_fn[T](y) = y
        op(identity_fn[T](x), identity_fn[T](y)) = op(x, y)
        identity_fn[T](op(x, y)) = op(identity_fn[T](x), identity_fn[T](y))
    }
}

/// Composition preserves unary-operation preservation.
theorem compose_preserves_unary_op[A, B, C](f: B -> C, g: A -> B, op_a: A -> A, op_b: B -> B, op_c: C -> C) {
    preserves_unary_op(f, op_b, op_c) and preserves_unary_op(g, op_a, op_b) implies
    preserves_unary_op(compose(f, g), op_a, op_c)
} by {
    if preserves_unary_op(f, op_b, op_c) and preserves_unary_op(g, op_a, op_b) {
        forall(x: A) {
            compose(f, g, op_a(x)) = f(g(op_a(x)))
            preserves_unary_op_step(g, op_a, op_b, x)
            g(op_a(x)) = op_b(g(x))
            compose(f, g, op_a(x)) = f(op_b(g(x)))
            preserves_unary_op_step(f, op_b, op_c, g(x))
            f(op_b(g(x))) = op_c(f(g(x)))
            compose(f, g, op_a(x)) = op_c(f(g(x)))
            compose(f, g, x) = f(g(x))
            op_c(compose(f, g)(x)) = op_c(f(g(x)))
            compose(f, g, op_a(x)) = op_c(compose(f, g)(x))
        }
    }
}

/// Composition preserves binary-operation preservation.
theorem compose_preserves_binary_op[A, B, C](
    f: B -> C,
    g: A -> B,
    op_a: (A, A) -> A,
    op_b: (B, B) -> B,
    op_c: (C, C) -> C
) {
    preserves_binary_op(f, op_b, op_c) and preserves_binary_op(g, op_a, op_b) implies
    preserves_binary_op(compose(f, g), op_a, op_c)
} by {
    if preserves_binary_op(f, op_b, op_c) and preserves_binary_op(g, op_a, op_b) {
        forall(x: A, y: A) {
            compose(f, g, op_a(x, y)) = f(g(op_a(x, y)))
            preserves_binary_op_step(g, op_a, op_b, x, y)
            g(op_a(x, y)) = op_b(g(x), g(y))
            compose(f, g, op_a(x, y)) = f(op_b(g(x), g(y)))
            preserves_binary_op_step(f, op_b, op_c, g(x), g(y))
            f(op_b(g(x), g(y))) = op_c(f(g(x)), f(g(y)))
            compose(f, g, op_a(x, y)) = op_c(f(g(x)), f(g(y)))
            compose(f, g, x) = f(g(x))
            compose(f, g, y) = f(g(y))
            op_c(compose(f, g)(x), compose(f, g)(y)) = op_c(f(g(x)), f(g(y)))
            compose(f, g, op_a(x, y)) = op_c(compose(f, g)(x), compose(f, g)(y))
        }
    }
}

/// Equality is compatible with every unary operation.
theorem eq_relation_respects_unary_op[T](op: T -> T) {
    respects_unary_op(eq_relation[T], op)
} by {
    forall(x: T, y: T) {
        if eq_relation[T](x, y) {
            eq_relation[T](x, y) = (x = y)
            x = y
            op(x) = op(y)
            eq_relation[T](op(x), op(y))
        }
    }
}

/// Equality is compatible with every binary operation.
theorem eq_relation_respects_binary_op[T](op: (T, T) -> T) {
    respects_binary_op(eq_relation[T], op)
} by {
    forall(x1: T, y1: T, x2: T, y2: T) {
        if eq_relation[T](x1, y1) and eq_relation[T](x2, y2) {
            eq_relation[T](x1, y1) = (x1 = y1)
            x1 = y1
            eq_relation[T](x2, y2) = (x2 = y2)
            x2 = y2
            op(x1, x2) = op(y1, y2)
            eq_relation[T](op(x1, x2), op(y1, y2))
        }
    }
}

/// Converse preserves unary compatibility.
theorem relation_converse_respects_unary_op[T](r: (T, T) -> Bool, op: T -> T) {
    respects_unary_op(r, op) implies respects_unary_op(relation_converse(r), op)
} by {
    if respects_unary_op(r, op) {
        forall(x: T, y: T) {
            if relation_converse(r, x, y) {
                relation_converse(r, x, y) = r(y, x)
                r(y, x)
                respects_unary_op_step(r, op, y, x)
                r(op(y), op(x))
                relation_converse(r, op(x), op(y)) = r(op(y), op(x))
                relation_converse(r, op(x), op(y))
            }
        }
    }
}

/// Converse preserves binary compatibility.
theorem relation_converse_respects_binary_op[T](r: (T, T) -> Bool, op: (T, T) -> T) {
    respects_binary_op(r, op) implies respects_binary_op(relation_converse(r), op)
} by {
    if respects_binary_op(r, op) {
        forall(x1: T, y1: T, x2: T, y2: T) {
            if relation_converse(r, x1, y1) and relation_converse(r, x2, y2) {
                relation_converse(r, x1, y1) = r(y1, x1)
                r(y1, x1)
                relation_converse(r, x2, y2) = r(y2, x2)
                r(y2, x2)
                respects_binary_op_step(r, op, y1, x1, y2, x2)
                r(op(y1, y2), op(x1, x2))
                relation_converse(r, op(x1, x2), op(y1, y2)) = r(op(y1, y2), op(x1, x2))
                relation_converse(r, op(x1, x2), op(y1, y2))
            }
        }
    }
}

/// Intersection preserves unary compatibility.
theorem relation_intersection_respects_unary_op[T](r: (T, T) -> Bool, s: (T, T) -> Bool, op: T -> T) {
    respects_unary_op(r, op) and respects_unary_op(s, op) implies
    respects_unary_op(relation_intersection(r, s), op)
} by {
    if respects_unary_op(r, op) and respects_unary_op(s, op) {
        forall(x: T, y: T) {
            if relation_intersection(r, s, x, y) {
                r(x, y)
                respects_unary_op_step(r, op, x, y)
                r(op(x), op(y))
                s(x, y)
                respects_unary_op_step(s, op, x, y)
                s(op(x), op(y))
                relation_intersection(r, s, op(x), op(y))
            }
        }
    }
}

/// Intersection preserves binary compatibility.
theorem relation_intersection_respects_binary_op[T](r: (T, T) -> Bool, s: (T, T) -> Bool, op: (T, T) -> T) {
    respects_binary_op(r, op) and respects_binary_op(s, op) implies
    respects_binary_op(relation_intersection(r, s), op)
} by {
    if respects_binary_op(r, op) and respects_binary_op(s, op) {
        forall(x1: T, y1: T, x2: T, y2: T) {
            if relation_intersection(r, s, x1, y1) and relation_intersection(r, s, x2, y2) {
                r(x1, y1)
                r(x2, y2)
                respects_binary_op_step(r, op, x1, y1, x2, y2)
                r(op(x1, x2), op(y1, y2))
                s(x1, y1)
                s(x2, y2)
                respects_binary_op_step(s, op, x1, y1, x2, y2)
                s(op(x1, x2), op(y1, y2))
                relation_intersection(r, s, op(x1, x2), op(y1, y2))
            }
        }
    }
}

/// Symmetric closure preserves unary compatibility.
theorem relation_symmetric_closure_respects_unary_op[T](r: (T, T) -> Bool, op: T -> T) {
    respects_unary_op(r, op) implies respects_unary_op(relation_symmetric_closure(r), op)
} by {
    if respects_unary_op(r, op) {
        relation_converse_respects_unary_op(r, op)
        respects_unary_op(relation_converse(r), op)
        forall(x: T, y: T) {
            if relation_symmetric_closure(r, x, y) {
                relation_symmetric_closure(r, x, y) = (r(x, y) or relation_converse(r, x, y))
                if r(x, y) {
                    respects_unary_op_step(r, op, x, y)
                    r(op(x), op(y))
                    relation_symmetric_closure(r, op(x), op(y))
                } else {
                    respects_unary_op_step(relation_converse(r), op, x, y)
                    relation_converse(r, op(x), op(y))
                    relation_symmetric_closure(r, op(x), op(y))
                }
            }
        }
    }
}

/// Reflexive closure preserves unary compatibility.
theorem relation_reflexive_closure_respects_unary_op[T](r: (T, T) -> Bool, op: T -> T) {
    respects_unary_op(r, op) implies respects_unary_op(relation_reflexive_closure(r), op)
} by {
    if respects_unary_op(r, op) {
        forall(x: T, y: T) {
            if relation_reflexive_closure(r, x, y) {
                relation_reflexive_closure(r, x, y) = (eq_relation[T](x, y) or r(x, y))
                if eq_relation[T](x, y) {
                    eq_relation[T](x, y) = (x = y)
                    x = y
                    op(x) = op(y)
                    eq_relation[T](op(x), op(y))
                    relation_reflexive_closure(r, op(x), op(y))
                } else {
                    respects_unary_op_step(r, op, x, y)
                    r(op(x), op(y))
                    relation_reflexive_closure(r, op(x), op(y))
                }
            }
        }
    }
}

/// Pullback preserves unary compatibility when the function preserves the operation.
theorem relation_pullback_respects_unary_op_of_preserves_unary_op[A, B](f: A -> B, r: (B, B) -> Bool, op_a: A -> A, op_b: B -> B) {
    respects_unary_op(r, op_b) and preserves_unary_op(f, op_a, op_b) implies
    respects_unary_op(relation_pullback(f, r), op_a)
} by {
    if respects_unary_op(r, op_b) and preserves_unary_op(f, op_a, op_b) {
        forall(x: A, y: A) {
            if relation_pullback(f, r, x, y) {
                relation_pullback(f, r, x, y) = r(f(x), f(y))
                r(f(x), f(y))
                respects_unary_op_step(r, op_b, f(x), f(y))
                r(op_b(f(x)), op_b(f(y)))
                preserves_unary_op_step(f, op_a, op_b, x)
                f(op_a(x)) = op_b(f(x))
                preserves_unary_op_step(f, op_a, op_b, y)
                f(op_a(y)) = op_b(f(y))
                relation_pullback(f, r, op_a(x), op_a(y)) = r(f(op_a(x)), f(op_a(y)))
                relation_pullback(f, r, op_a(x), op_a(y))
            }
        }
    }
}

/// Pullback preserves binary compatibility when the function preserves the operation.
theorem relation_pullback_respects_binary_op_of_preserves_binary_op[A, B](
    f: A -> B,
    r: (B, B) -> Bool,
    op_a: (A, A) -> A,
    op_b: (B, B) -> B
) {
    respects_binary_op(r, op_b) and preserves_binary_op(f, op_a, op_b) implies
    respects_binary_op(relation_pullback(f, r), op_a)
} by {
    if respects_binary_op(r, op_b) and preserves_binary_op(f, op_a, op_b) {
        forall(x1: A, y1: A, x2: A, y2: A) {
            if relation_pullback(f, r, x1, y1) and relation_pullback(f, r, x2, y2) {
                relation_pullback(f, r, x1, y1) = r(f(x1), f(y1))
                r(f(x1), f(y1))
                relation_pullback(f, r, x2, y2) = r(f(x2), f(y2))
                r(f(x2), f(y2))
                respects_binary_op_step(r, op_b, f(x1), f(y1), f(x2), f(y2))
                r(op_b(f(x1), f(x2)), op_b(f(y1), f(y2)))
                preserves_binary_op_step(f, op_a, op_b, x1, x2)
                f(op_a(x1, x2)) = op_b(f(x1), f(x2))
                preserves_binary_op_step(f, op_a, op_b, y1, y2)
                f(op_a(y1, y2)) = op_b(f(y1), f(y2))
                relation_pullback(f, r, op_a(x1, x2), op_a(y1, y2)) = r(f(op_a(x1, x2)), f(op_a(y1, y2)))
                relation_pullback(f, r, op_a(x1, x2), op_a(y1, y2))
            }
        }
    }
}

/// Surjective pullback reflects unary compatibility for operation-preserving maps.
theorem relation_pullback_reflects_respects_unary_op_of_surjective[A, B](
    f: A -> B,
    r: (B, B) -> Bool,
    op_a: A -> A,
    op_b: B -> B
) {
    is_surjective_fn(f) and respects_unary_op(relation_pullback(f, r), op_a) and preserves_unary_op(f, op_a, op_b) implies
    respects_unary_op(r, op_b)
} by {
    if is_surjective_fn(f) and respects_unary_op(relation_pullback(f, r), op_a) and preserves_unary_op(f, op_a, op_b) {
        forall(y1: B, y2: B) {
            if r(y1, y2) {
                surjective_fn_has_preimage(f, y1)
                let x1: A satisfy {
                    f(x1) = y1
                }
                surjective_fn_has_preimage(f, y2)
                let x2: A satisfy {
                    f(x2) = y2
                }
                relation_pullback(f, r, x1, x2) = r(f(x1), f(x2))
                f(x1) = y1
                f(x2) = y2
                relation_pullback(f, r, x1, x2)
                respects_unary_op_step(relation_pullback(f, r), op_a, x1, x2)
                relation_pullback(f, r, op_a(x1), op_a(x2))
                relation_pullback(f, r, op_a(x1), op_a(x2)) = r(f(op_a(x1)), f(op_a(x2)))
                preserves_unary_op_step(f, op_a, op_b, x1)
                f(op_a(x1)) = op_b(f(x1))
                preserves_unary_op_step(f, op_a, op_b, x2)
                f(op_a(x2)) = op_b(f(x2))
                r(op_b(y1), op_b(y2))
            }
        }
    }
}

/// Surjective pullback reflects binary compatibility for operation-preserving maps.
theorem relation_pullback_reflects_respects_binary_op_of_surjective[A, B](
    f: A -> B,
    r: (B, B) -> Bool,
    op_a: (A, A) -> A,
    op_b: (B, B) -> B
) {
    is_surjective_fn(f) and respects_binary_op(relation_pullback(f, r), op_a) and preserves_binary_op(f, op_a, op_b) implies
    respects_binary_op(r, op_b)
} by {
    if is_surjective_fn(f) and respects_binary_op(relation_pullback(f, r), op_a) and preserves_binary_op(f, op_a, op_b) {
        forall(y1: B, z1: B, y2: B, z2: B) {
            if r(y1, z1) and r(y2, z2) {
                surjective_fn_has_preimage(f, y1)
                let x1: A satisfy {
                    f(x1) = y1
                }
                surjective_fn_has_preimage(f, z1)
                let w1: A satisfy {
                    f(w1) = z1
                }
                surjective_fn_has_preimage(f, y2)
                let x2: A satisfy {
                    f(x2) = y2
                }
                surjective_fn_has_preimage(f, z2)
                let w2: A satisfy {
                    f(w2) = z2
                }
                relation_pullback(f, r, x1, w1) = r(f(x1), f(w1))
                f(x1) = y1
                f(w1) = z1
                relation_pullback(f, r, x1, w1)
                relation_pullback(f, r, x2, w2) = r(f(x2), f(w2))
                f(x2) = y2
                f(w2) = z2
                relation_pullback(f, r, x2, w2)
                respects_binary_op_step(relation_pullback(f, r), op_a, x1, w1, x2, w2)
                relation_pullback(f, r, op_a(x1, x2), op_a(w1, w2))
                relation_pullback(f, r, op_a(x1, x2), op_a(w1, w2)) = r(f(op_a(x1, x2)), f(op_a(w1, w2)))
                preserves_binary_op_step(f, op_a, op_b, x1, x2)
                f(op_a(x1, x2)) = op_b(f(x1), f(x2))
                preserves_binary_op_step(f, op_a, op_b, w1, w2)
                f(op_a(w1, w2)) = op_b(f(w1), f(w2))
                r(op_b(y1, y2), op_b(z1, z2))
            }
        }
    }
}

/// Surjective pullback exactly detects unary compatibility for operation-preserving maps.
theorem relation_pullback_respects_unary_op_iff_of_surjective[A, B](
    f: A -> B,
    r: (B, B) -> Bool,
    op_a: A -> A,
    op_b: B -> B
) {
    is_surjective_fn(f) and preserves_unary_op(f, op_a, op_b) implies
    respects_unary_op(relation_pullback(f, r), op_a) = respects_unary_op(r, op_b)
} by {
    if is_surjective_fn(f) and preserves_unary_op(f, op_a, op_b) {
        if respects_unary_op(relation_pullback(f, r), op_a) {
            relation_pullback_reflects_respects_unary_op_of_surjective(f, r, op_a, op_b)
            respects_unary_op(r, op_b)
        }
        if respects_unary_op(r, op_b) {
            relation_pullback_respects_unary_op_of_preserves_unary_op(f, r, op_a, op_b)
            respects_unary_op(relation_pullback(f, r), op_a)
        }
        respects_unary_op(relation_pullback(f, r), op_a) = respects_unary_op(r, op_b)
    }
}

/// Surjective pullback exactly detects binary compatibility for operation-preserving maps.
theorem relation_pullback_respects_binary_op_iff_of_surjective[A, B](
    f: A -> B,
    r: (B, B) -> Bool,
    op_a: (A, A) -> A,
    op_b: (B, B) -> B
) {
    is_surjective_fn(f) and preserves_binary_op(f, op_a, op_b) implies
    respects_binary_op(relation_pullback(f, r), op_a) = respects_binary_op(r, op_b)
} by {
    if is_surjective_fn(f) and preserves_binary_op(f, op_a, op_b) {
        if respects_binary_op(relation_pullback(f, r), op_a) {
            relation_pullback_reflects_respects_binary_op_of_surjective(f, r, op_a, op_b)
            respects_binary_op(r, op_b)
        }
        if respects_binary_op(r, op_b) {
            relation_pullback_respects_binary_op_of_preserves_binary_op(f, r, op_a, op_b)
            respects_binary_op(relation_pullback(f, r), op_a)
        }
        respects_binary_op(relation_pullback(f, r), op_a) = respects_binary_op(r, op_b)
    }
}

/// The equality pullback of a unary-operation-preserving map is unary-compatible.
theorem relation_pullback_eq_relation_respects_unary_op[A, B](f: A -> B, op_a: A -> A, op_b: B -> B) {
    preserves_unary_op(f, op_a, op_b) implies respects_unary_op(relation_pullback(f, eq_relation[B]), op_a)
} by {
    if preserves_unary_op(f, op_a, op_b) {
        eq_relation_respects_unary_op(op_b)
        relation_pullback_respects_unary_op_of_preserves_unary_op(f, eq_relation[B], op_a, op_b)
        respects_unary_op(relation_pullback(f, eq_relation[B]), op_a)
    }
}

/// The equality pullback of a binary-operation-preserving map is binary-compatible.
theorem relation_pullback_eq_relation_respects_binary_op[A, B](f: A -> B, op_a: (A, A) -> A, op_b: (B, B) -> B) {
    preserves_binary_op(f, op_a, op_b) implies respects_binary_op(relation_pullback(f, eq_relation[B]), op_a)
} by {
    if preserves_binary_op(f, op_a, op_b) {
        eq_relation_respects_binary_op(op_b)
        relation_pullback_respects_binary_op_of_preserves_binary_op(f, eq_relation[B], op_a, op_b)
        respects_binary_op(relation_pullback(f, eq_relation[B]), op_a)
    }
}

/// A unary congruence relation is in particular an equivalence relation.
theorem unary_congruence_is_equivalence[T](r: (T, T) -> Bool, op: T -> T) {
    is_unary_congruence(r, op) implies is_equivalence(r)
} by {
    if is_unary_congruence(r, op) {
        is_unary_congruence(r, op) = (is_equivalence(r) and respects_unary_op(r, op))
        is_equivalence(r)
    }
}

/// A unary congruence relation is compatible with its unary operation.
theorem unary_congruence_respects_unary_op[T](r: (T, T) -> Bool, op: T -> T) {
    is_unary_congruence(r, op) implies respects_unary_op(r, op)
} by {
    if is_unary_congruence(r, op) {
        is_unary_congruence(r, op) = (is_equivalence(r) and respects_unary_op(r, op))
        respects_unary_op(r, op)
    }
}

/// A binary congruence relation is in particular an equivalence relation.
theorem binary_congruence_is_equivalence[T](r: (T, T) -> Bool, op: (T, T) -> T) {
    is_binary_congruence(r, op) implies is_equivalence(r)
} by {
    if is_binary_congruence(r, op) {
        is_binary_congruence(r, op) = (is_equivalence(r) and respects_binary_op(r, op))
        is_equivalence(r)
    }
}

/// A binary congruence relation is compatible with its binary operation.
theorem binary_congruence_respects_binary_op[T](r: (T, T) -> Bool, op: (T, T) -> T) {
    is_binary_congruence(r, op) implies respects_binary_op(r, op)
} by {
    if is_binary_congruence(r, op) {
        is_binary_congruence(r, op) = (is_equivalence(r) and respects_binary_op(r, op))
        respects_binary_op(r, op)
    }
}

/// A unary congruence relation is reflexive.
theorem unary_congruence_is_reflexive[T](r: (T, T) -> Bool, op: T -> T) {
    is_unary_congruence(r, op) implies is_reflexive(r)
} by {
    if is_unary_congruence(r, op) {
        unary_congruence_is_equivalence(r, op)
        is_equivalence(r)
        equivalence_is_reflexive(r)
        is_reflexive(r)
    }
}

/// A unary congruence relation is symmetric.
theorem unary_congruence_is_symmetric[T](r: (T, T) -> Bool, op: T -> T) {
    is_unary_congruence(r, op) implies is_symmetric(r)
} by {
    if is_unary_congruence(r, op) {
        unary_congruence_is_equivalence(r, op)
        is_equivalence(r)
        equivalence_is_symmetric(r)
        is_symmetric(r)
    }
}

/// A unary congruence relation is transitive.
theorem unary_congruence_is_transitive[T](r: (T, T) -> Bool, op: T -> T) {
    is_unary_congruence(r, op) implies is_transitive(r)
} by {
    if is_unary_congruence(r, op) {
        unary_congruence_is_equivalence(r, op)
        is_equivalence(r)
        equivalence_is_transitive(r)
        is_transitive(r)
    }
}

/// A binary congruence relation is reflexive.
theorem binary_congruence_is_reflexive[T](r: (T, T) -> Bool, op: (T, T) -> T) {
    is_binary_congruence(r, op) implies is_reflexive(r)
} by {
    if is_binary_congruence(r, op) {
        binary_congruence_is_equivalence(r, op)
        is_equivalence(r)
        equivalence_is_reflexive(r)
        is_reflexive(r)
    }
}

/// A binary congruence relation is symmetric.
theorem binary_congruence_is_symmetric[T](r: (T, T) -> Bool, op: (T, T) -> T) {
    is_binary_congruence(r, op) implies is_symmetric(r)
} by {
    if is_binary_congruence(r, op) {
        binary_congruence_is_equivalence(r, op)
        is_equivalence(r)
        equivalence_is_symmetric(r)
        is_symmetric(r)
    }
}

/// A binary congruence relation is transitive.
theorem binary_congruence_is_transitive[T](r: (T, T) -> Bool, op: (T, T) -> T) {
    is_binary_congruence(r, op) implies is_transitive(r)
} by {
    if is_binary_congruence(r, op) {
        binary_congruence_is_equivalence(r, op)
        is_equivalence(r)
        equivalence_is_transitive(r)
        is_transitive(r)
    }
}

/// A unary congruence relation may be applied at any related pair.
theorem unary_congruence_step[T](r: (T, T) -> Bool, op: T -> T, x: T, y: T) {
    is_unary_congruence(r, op) and r(x, y) implies r(op(x), op(y))
} by {
    if is_unary_congruence(r, op) and r(x, y) {
        unary_congruence_respects_unary_op(r, op)
        respects_unary_op(r, op)
        respects_unary_op_step(r, op, x, y)
        r(op(x), op(y))
    }
}

/// A binary congruence relation may be applied at any pair of related inputs.
theorem binary_congruence_step[T](r: (T, T) -> Bool, op: (T, T) -> T, x1: T, y1: T, x2: T, y2: T) {
    is_binary_congruence(r, op) and r(x1, y1) and r(x2, y2) implies r(op(x1, x2), op(y1, y2))
} by {
    if is_binary_congruence(r, op) and r(x1, y1) and r(x2, y2) {
        binary_congruence_respects_binary_op(r, op)
        respects_binary_op(r, op)
        respects_binary_op_step(r, op, x1, y1, x2, y2)
        r(op(x1, x2), op(y1, y2))
    }
}

/// A unary congruence relation relates every element to itself.
theorem unary_congruence_self[T](r: (T, T) -> Bool, op: T -> T, x: T) {
    is_unary_congruence(r, op) implies r(x, x)
} by {
    if is_unary_congruence(r, op) {
        unary_congruence_is_reflexive(r, op)
        reflexive_self(r, x)
        r(x, x)
    }
}

/// A binary congruence relation relates every element to itself.
theorem binary_congruence_self[T](r: (T, T) -> Bool, op: (T, T) -> T, x: T) {
    is_binary_congruence(r, op) implies r(x, x)
} by {
    if is_binary_congruence(r, op) {
        binary_congruence_is_reflexive(r, op)
        reflexive_self(r, x)
        r(x, x)
    }
}

/// A unary congruence relation is symmetric at each related pair.
theorem unary_congruence_flip[T](r: (T, T) -> Bool, op: T -> T, x: T, y: T) {
    is_unary_congruence(r, op) and r(x, y) implies r(y, x)
} by {
    if is_unary_congruence(r, op) and r(x, y) {
        unary_congruence_is_symmetric(r, op)
        symmetric_flip(r, x, y)
        r(y, x)
    }
}

/// A binary congruence relation is symmetric at each related pair.
theorem binary_congruence_flip[T](r: (T, T) -> Bool, op: (T, T) -> T, x: T, y: T) {
    is_binary_congruence(r, op) and r(x, y) implies r(y, x)
} by {
    if is_binary_congruence(r, op) and r(x, y) {
        binary_congruence_is_symmetric(r, op)
        symmetric_flip(r, x, y)
        r(y, x)
    }
}

/// Equality is always a unary congruence relation.
theorem eq_relation_is_unary_congruence[T](op: T -> T) {
    is_unary_congruence(eq_relation[T], op)
} by {
    eq_relation_is_equivalence[T]
    is_equivalence(eq_relation[T])
    eq_relation_respects_unary_op(op)
    respects_unary_op(eq_relation[T], op)
    is_unary_congruence(eq_relation[T], op)
}

/// Equality is always a binary congruence relation.
theorem eq_relation_is_binary_congruence[T](op: (T, T) -> T) {
    is_binary_congruence(eq_relation[T], op)
} by {
    eq_relation_is_equivalence[T]
    is_equivalence(eq_relation[T])
    eq_relation_respects_binary_op(op)
    respects_binary_op(eq_relation[T], op)
    is_binary_congruence(eq_relation[T], op)
}

/// Converse preserves unary congruence relations.
theorem relation_converse_is_unary_congruence[T](r: (T, T) -> Bool, op: T -> T) {
    is_unary_congruence(r, op) implies is_unary_congruence(relation_converse(r), op)
} by {
    if is_unary_congruence(r, op) {
        unary_congruence_is_equivalence(r, op)
        is_equivalence(r)
        relation_converse_is_equivalence(r)
        is_equivalence(relation_converse(r))
        unary_congruence_respects_unary_op(r, op)
        respects_unary_op(r, op)
        relation_converse_respects_unary_op(r, op)
        respects_unary_op(relation_converse(r), op)
        is_unary_congruence(relation_converse(r), op)
    }
}

/// Converse preserves binary congruence relations.
theorem relation_converse_is_binary_congruence[T](r: (T, T) -> Bool, op: (T, T) -> T) {
    is_binary_congruence(r, op) implies is_binary_congruence(relation_converse(r), op)
} by {
    if is_binary_congruence(r, op) {
        binary_congruence_is_equivalence(r, op)
        is_equivalence(r)
        relation_converse_is_equivalence(r)
        is_equivalence(relation_converse(r))
        binary_congruence_respects_binary_op(r, op)
        respects_binary_op(r, op)
        relation_converse_respects_binary_op(r, op)
        respects_binary_op(relation_converse(r), op)
        is_binary_congruence(relation_converse(r), op)
    }
}

/// Intersection preserves unary congruence relations.
theorem relation_intersection_is_unary_congruence[T](r: (T, T) -> Bool, s: (T, T) -> Bool, op: T -> T) {
    is_unary_congruence(r, op) and is_unary_congruence(s, op) implies
    is_unary_congruence(relation_intersection(r, s), op)
} by {
    if is_unary_congruence(r, op) and is_unary_congruence(s, op) {
        unary_congruence_is_equivalence(r, op)
        is_equivalence(r)
        unary_congruence_is_equivalence(s, op)
        is_equivalence(s)
        relation_intersection_is_equivalence(r, s)
        is_equivalence(relation_intersection(r, s))
        unary_congruence_respects_unary_op(r, op)
        respects_unary_op(r, op)
        unary_congruence_respects_unary_op(s, op)
        respects_unary_op(s, op)
        relation_intersection_respects_unary_op(r, s, op)
        respects_unary_op(relation_intersection(r, s), op)
        is_unary_congruence(relation_intersection(r, s), op)
    }
}

/// Intersection preserves binary congruence relations.
theorem relation_intersection_is_binary_congruence[T](r: (T, T) -> Bool, s: (T, T) -> Bool, op: (T, T) -> T) {
    is_binary_congruence(r, op) and is_binary_congruence(s, op) implies
    is_binary_congruence(relation_intersection(r, s), op)
} by {
    if is_binary_congruence(r, op) and is_binary_congruence(s, op) {
        binary_congruence_is_equivalence(r, op)
        is_equivalence(r)
        binary_congruence_is_equivalence(s, op)
        is_equivalence(s)
        relation_intersection_is_equivalence(r, s)
        is_equivalence(relation_intersection(r, s))
        binary_congruence_respects_binary_op(r, op)
        respects_binary_op(r, op)
        binary_congruence_respects_binary_op(s, op)
        respects_binary_op(s, op)
        relation_intersection_respects_binary_op(r, s, op)
        respects_binary_op(relation_intersection(r, s), op)
        is_binary_congruence(relation_intersection(r, s), op)
    }
}

/// Pullback preserves unary congruence relations along unary-operation-preserving maps.
theorem relation_pullback_is_unary_congruence_of_preserves_unary_op[A, B](
    f: A -> B,
    r: (B, B) -> Bool,
    op_a: A -> A,
    op_b: B -> B
) {
    is_unary_congruence(r, op_b) and preserves_unary_op(f, op_a, op_b) implies
    is_unary_congruence(relation_pullback(f, r), op_a)
} by {
    if is_unary_congruence(r, op_b) and preserves_unary_op(f, op_a, op_b) {
        unary_congruence_is_equivalence(r, op_b)
        is_equivalence(r)
        relation_pullback_is_equivalence(f, r)
        is_equivalence(relation_pullback(f, r))
        unary_congruence_respects_unary_op(r, op_b)
        respects_unary_op(r, op_b)
        relation_pullback_respects_unary_op_of_preserves_unary_op(f, r, op_a, op_b)
        respects_unary_op(relation_pullback(f, r), op_a)
        is_unary_congruence(relation_pullback(f, r), op_a)
    }
}

/// Pullback preserves binary congruence relations along binary-operation-preserving maps.
theorem relation_pullback_is_binary_congruence_of_preserves_binary_op[A, B](
    f: A -> B,
    r: (B, B) -> Bool,
    op_a: (A, A) -> A,
    op_b: (B, B) -> B
) {
    is_binary_congruence(r, op_b) and preserves_binary_op(f, op_a, op_b) implies
    is_binary_congruence(relation_pullback(f, r), op_a)
} by {
    if is_binary_congruence(r, op_b) and preserves_binary_op(f, op_a, op_b) {
        binary_congruence_is_equivalence(r, op_b)
        is_equivalence(r)
        relation_pullback_is_equivalence(f, r)
        is_equivalence(relation_pullback(f, r))
        binary_congruence_respects_binary_op(r, op_b)
        respects_binary_op(r, op_b)
        relation_pullback_respects_binary_op_of_preserves_binary_op(f, r, op_a, op_b)
        respects_binary_op(relation_pullback(f, r), op_a)
        is_binary_congruence(relation_pullback(f, r), op_a)
    }
}

/// Surjective pullback reflects unary congruence relations along unary-operation-preserving maps.
theorem relation_pullback_reflects_unary_congruence_of_surjective[A, B](
    f: A -> B,
    r: (B, B) -> Bool,
    op_a: A -> A,
    op_b: B -> B
) {
    is_surjective_fn(f) and is_unary_congruence(relation_pullback(f, r), op_a) and preserves_unary_op(f, op_a, op_b) implies
    is_unary_congruence(r, op_b)
} by {
    if is_surjective_fn(f) and is_unary_congruence(relation_pullback(f, r), op_a) and preserves_unary_op(f, op_a, op_b) {
        relation_pullback_is_equivalence_iff_of_surjective(f, r)
        is_equivalence(relation_pullback(f, r)) = is_equivalence(r)
        unary_congruence_is_equivalence(relation_pullback(f, r), op_a)
        is_equivalence(relation_pullback(f, r))
        is_equivalence(r)
        unary_congruence_respects_unary_op(relation_pullback(f, r), op_a)
        respects_unary_op(relation_pullback(f, r), op_a)
        relation_pullback_reflects_respects_unary_op_of_surjective(f, r, op_a, op_b)
        respects_unary_op(r, op_b)
        is_unary_congruence(r, op_b)
    }
}

/// Surjective pullback reflects binary congruence relations along binary-operation-preserving maps.
theorem relation_pullback_reflects_binary_congruence_of_surjective[A, B](
    f: A -> B,
    r: (B, B) -> Bool,
    op_a: (A, A) -> A,
    op_b: (B, B) -> B
) {
    is_surjective_fn(f) and is_binary_congruence(relation_pullback(f, r), op_a) and preserves_binary_op(f, op_a, op_b) implies
    is_binary_congruence(r, op_b)
} by {
    if is_surjective_fn(f) and is_binary_congruence(relation_pullback(f, r), op_a) and preserves_binary_op(f, op_a, op_b) {
        relation_pullback_is_equivalence_iff_of_surjective(f, r)
        is_equivalence(relation_pullback(f, r)) = is_equivalence(r)
        binary_congruence_is_equivalence(relation_pullback(f, r), op_a)
        is_equivalence(relation_pullback(f, r))
        is_equivalence(r)
        binary_congruence_respects_binary_op(relation_pullback(f, r), op_a)
        respects_binary_op(relation_pullback(f, r), op_a)
        relation_pullback_reflects_respects_binary_op_of_surjective(f, r, op_a, op_b)
        respects_binary_op(r, op_b)
        is_binary_congruence(r, op_b)
    }
}

/// Surjective pullback exactly detects unary congruence relations along unary-operation-preserving maps.
theorem relation_pullback_is_unary_congruence_iff_of_surjective[A, B](
    f: A -> B,
    r: (B, B) -> Bool,
    op_a: A -> A,
    op_b: B -> B
) {
    is_surjective_fn(f) and preserves_unary_op(f, op_a, op_b) implies
    is_unary_congruence(relation_pullback(f, r), op_a) = is_unary_congruence(r, op_b)
} by {
    if is_surjective_fn(f) and preserves_unary_op(f, op_a, op_b) {
        if is_unary_congruence(relation_pullback(f, r), op_a) {
            relation_pullback_reflects_unary_congruence_of_surjective(f, r, op_a, op_b)
            is_unary_congruence(r, op_b)
        }
        if is_unary_congruence(r, op_b) {
            relation_pullback_is_unary_congruence_of_preserves_unary_op(f, r, op_a, op_b)
            is_unary_congruence(relation_pullback(f, r), op_a)
        }
        is_unary_congruence(relation_pullback(f, r), op_a) = is_unary_congruence(r, op_b)
    }
}

/// Surjective pullback exactly detects binary congruence relations along binary-operation-preserving maps.
theorem relation_pullback_is_binary_congruence_iff_of_surjective[A, B](
    f: A -> B,
    r: (B, B) -> Bool,
    op_a: (A, A) -> A,
    op_b: (B, B) -> B
) {
    is_surjective_fn(f) and preserves_binary_op(f, op_a, op_b) implies
    is_binary_congruence(relation_pullback(f, r), op_a) = is_binary_congruence(r, op_b)
} by {
    if is_surjective_fn(f) and preserves_binary_op(f, op_a, op_b) {
        if is_binary_congruence(relation_pullback(f, r), op_a) {
            relation_pullback_reflects_binary_congruence_of_surjective(f, r, op_a, op_b)
            is_binary_congruence(r, op_b)
        }
        if is_binary_congruence(r, op_b) {
            relation_pullback_is_binary_congruence_of_preserves_binary_op(f, r, op_a, op_b)
            is_binary_congruence(relation_pullback(f, r), op_a)
        }
        is_binary_congruence(relation_pullback(f, r), op_a) = is_binary_congruence(r, op_b)
    }
}

/// The equality pullback of a unary-operation-preserving map is a unary congruence relation.
theorem relation_pullback_eq_relation_is_unary_congruence[A, B](f: A -> B, op_a: A -> A, op_b: B -> B) {
    preserves_unary_op(f, op_a, op_b) implies
    is_unary_congruence(relation_pullback(f, eq_relation[B]), op_a)
} by {
    if preserves_unary_op(f, op_a, op_b) {
        eq_relation_is_unary_congruence(op_b)
        is_unary_congruence(eq_relation[B], op_b)
        relation_pullback_is_unary_congruence_of_preserves_unary_op(f, eq_relation[B], op_a, op_b)
        is_unary_congruence(relation_pullback(f, eq_relation[B]), op_a)
    }
}

/// The equality pullback of a binary-operation-preserving map is a binary congruence relation.
theorem relation_pullback_eq_relation_is_binary_congruence[A, B](f: A -> B, op_a: (A, A) -> A, op_b: (B, B) -> B) {
    preserves_binary_op(f, op_a, op_b) implies
    is_binary_congruence(relation_pullback(f, eq_relation[B]), op_a)
} by {
    if preserves_binary_op(f, op_a, op_b) {
        eq_relation_is_binary_congruence(op_b)
        is_binary_congruence(eq_relation[B], op_b)
        relation_pullback_is_binary_congruence_of_preserves_binary_op(f, eq_relation[B], op_a, op_b)
        is_binary_congruence(relation_pullback(f, eq_relation[B]), op_a)
    }
}

/// The unary operation on a target domain induced by a bijection from a source domain.
define unary_op_pushforward[A: Inhabited, B](f: A -> B, op: A -> A, y: B) -> B {
    f(op(inverse_fn(f, y)))
}

/// The binary operation on a target domain induced by a bijection from a source domain.
define binary_op_pushforward[A: Inhabited, B](f: A -> B, op: (A, A) -> A, y: B, z: B) -> B {
    f(op(inverse_fn(f, y), inverse_fn(f, z)))
}

/// Transporting a unary operation across a bijection agrees with the source operation on images.
theorem unary_op_pushforward_apply_image_of_bijection[A: Inhabited, B](f: A -> B, op: A -> A, x: A) {
    is_bijection_fn(f) implies unary_op_pushforward(f, op, f(x)) = f(op(x))
} by {
    if is_bijection_fn(f) {
        unary_op_pushforward(f, op, f(x)) = f(op(inverse_fn(f, f(x))))
        inverse_fn_apply_image_of_bijection(f, x)
        inverse_fn(f, f(x)) = x
        unary_op_pushforward(f, op, f(x)) = f(op(x))
    }
}

/// Transporting a binary operation across a bijection agrees with the source operation on images.
theorem binary_op_pushforward_apply_image_of_bijection[A: Inhabited, B](f: A -> B, op: (A, A) -> A, x: A, y: A) {
    is_bijection_fn(f) implies binary_op_pushforward(f, op, f(x), f(y)) = f(op(x, y))
} by {
    if is_bijection_fn(f) {
        binary_op_pushforward(f, op, f(x), f(y)) = f(op(inverse_fn(f, f(x)), inverse_fn(f, f(y))))
        inverse_fn_apply_image_of_bijection(f, x)
        inverse_fn_apply_image_of_bijection(f, y)
        inverse_fn(f, f(x)) = x
        inverse_fn(f, f(y)) = y
        binary_op_pushforward(f, op, f(x), f(y)) = f(op(x, y))
    }
}

/// A bijection preserves every unary operation after pushing that operation forward.
theorem bijection_preserves_unary_op_pushforward[A: Inhabited, B](f: A -> B, op: A -> A) {
    is_bijection_fn(f) implies preserves_unary_op(f, op, unary_op_pushforward(f, op))
} by {
    if is_bijection_fn(f) {
        forall(x: A) {
            unary_op_pushforward_apply_image_of_bijection(f, op, x)
            unary_op_pushforward(f, op, f(x)) = f(op(x))
            f(op(x)) = unary_op_pushforward(f, op, f(x))
        }
        preserves_unary_op(f, op, unary_op_pushforward(f, op))
    }
}

/// A bijection preserves every binary operation after pushing that operation forward.
theorem bijection_preserves_binary_op_pushforward[A: Inhabited, B](f: A -> B, op: (A, A) -> A) {
    is_bijection_fn(f) implies preserves_binary_op(f, op, binary_op_pushforward(f, op))
} by {
    if is_bijection_fn(f) {
        forall(x: A, y: A) {
            binary_op_pushforward_apply_image_of_bijection(f, op, x, y)
            binary_op_pushforward(f, op, f(x), f(y)) = f(op(x, y))
            f(op(x, y)) = binary_op_pushforward(f, op, f(x), f(y))
        }
        preserves_binary_op(f, op, binary_op_pushforward(f, op))
    }
}

/// Pushforward preserves unary compatibility along bijective transport of the operation.
theorem relation_pushforward_respects_unary_op_of_bijection[A: Inhabited, B](
    f: A -> B,
    r: (A, A) -> Bool,
    op: A -> A
) {
    is_bijection_fn(f) and respects_unary_op(r, op) implies
    respects_unary_op(relation_pushforward(f, r), unary_op_pushforward(f, op))
} by {
    if is_bijection_fn(f) and respects_unary_op(r, op) {
        bijection_fn_is_injective(f)
        is_injective_fn(f)
        bijection_fn_is_surjective(f)
        is_surjective_fn(f)
        forall(u: B, v: B) {
            if relation_pushforward(f, r, u, v) {
                relation_pushforward_has_preimages(f, r, u, v)
                let x: A satisfy {
                    exists(y: A) {
                        f(x) = u and f(y) = v and r(x, y)
                    }
                }
                let y: A satisfy {
                    f(x) = u and f(y) = v and r(x, y)
                }
                inverse_fn_apply_of_surjective(f, u)
                f(inverse_fn(f, u)) = u
                f(x) = u
                f(inverse_fn(f, u)) = f(x)
                injective_fn_eq(f, inverse_fn(f, u), x)
                inverse_fn(f, u) = x
                inverse_fn_apply_of_surjective(f, v)
                f(inverse_fn(f, v)) = v
                f(y) = v
                f(inverse_fn(f, v)) = f(y)
                injective_fn_eq(f, inverse_fn(f, v), y)
                inverse_fn(f, v) = y
                respects_unary_op_step(r, op, x, y)
                r(op(x), op(y))
                unary_op_pushforward(f, op, u) = f(op(inverse_fn(f, u)))
                unary_op_pushforward(f, op, u) = f(op(x))
                unary_op_pushforward(f, op, v) = f(op(inverse_fn(f, v)))
                unary_op_pushforward(f, op, v) = f(op(y))
                relation_pushforward_intro(f, r, op(x), op(y))
                relation_pushforward(f, r, f(op(x)), f(op(y)))
                relation_pushforward(f, r, unary_op_pushforward(f, op, u), unary_op_pushforward(f, op, v))
            }
        }
    }
}

/// Pushforward preserves binary compatibility along bijective transport of the operation.
theorem relation_pushforward_respects_binary_op_of_bijection[A: Inhabited, B](
    f: A -> B,
    r: (A, A) -> Bool,
    op: (A, A) -> A
) {
    is_bijection_fn(f) and respects_binary_op(r, op) implies
    respects_binary_op(relation_pushforward(f, r), binary_op_pushforward(f, op))
} by {
    if is_bijection_fn(f) and respects_binary_op(r, op) {
        bijection_fn_is_injective(f)
        is_injective_fn(f)
        bijection_fn_is_surjective(f)
        is_surjective_fn(f)
        forall(u1: B, v1: B, u2: B, v2: B) {
            if relation_pushforward(f, r, u1, v1) and relation_pushforward(f, r, u2, v2) {
                relation_pushforward_has_preimages(f, r, u1, v1)
                let x1: A satisfy {
                    exists(y1: A) {
                        f(x1) = u1 and f(y1) = v1 and r(x1, y1)
                    }
                }
                let y1: A satisfy {
                    f(x1) = u1 and f(y1) = v1 and r(x1, y1)
                }
                relation_pushforward_has_preimages(f, r, u2, v2)
                let x2: A satisfy {
                    exists(y2: A) {
                        f(x2) = u2 and f(y2) = v2 and r(x2, y2)
                    }
                }
                let y2: A satisfy {
                    f(x2) = u2 and f(y2) = v2 and r(x2, y2)
                }
                inverse_fn_apply_of_surjective(f, u1)
                f(inverse_fn(f, u1)) = u1
                f(inverse_fn(f, u1)) = f(x1)
                injective_fn_eq(f, inverse_fn(f, u1), x1)
                inverse_fn(f, u1) = x1
                inverse_fn_apply_of_surjective(f, v1)
                f(inverse_fn(f, v1)) = v1
                f(inverse_fn(f, v1)) = f(y1)
                injective_fn_eq(f, inverse_fn(f, v1), y1)
                inverse_fn(f, v1) = y1
                inverse_fn_apply_of_surjective(f, u2)
                f(inverse_fn(f, u2)) = u2
                f(inverse_fn(f, u2)) = f(x2)
                injective_fn_eq(f, inverse_fn(f, u2), x2)
                inverse_fn(f, u2) = x2
                inverse_fn_apply_of_surjective(f, v2)
                f(inverse_fn(f, v2)) = v2
                f(inverse_fn(f, v2)) = f(y2)
                injective_fn_eq(f, inverse_fn(f, v2), y2)
                inverse_fn(f, v2) = y2
                respects_binary_op_step(r, op, x1, y1, x2, y2)
                r(op(x1, x2), op(y1, y2))
                binary_op_pushforward(f, op, u1, u2) = f(op(inverse_fn(f, u1), inverse_fn(f, u2)))
                binary_op_pushforward(f, op, u1, u2) = f(op(x1, x2))
                binary_op_pushforward(f, op, v1, v2) = f(op(inverse_fn(f, v1), inverse_fn(f, v2)))
                binary_op_pushforward(f, op, v1, v2) = f(op(y1, y2))
                relation_pushforward_intro(f, r, op(x1, x2), op(y1, y2))
                relation_pushforward(f, r, f(op(x1, x2)), f(op(y1, y2)))
                relation_pushforward(f, r, binary_op_pushforward(f, op, u1, u2), binary_op_pushforward(f, op, v1, v2))
            }
        }
    }
}

/// Pushforward preserves unary congruence relations along bijective transport of the operation.
theorem relation_pushforward_is_unary_congruence_of_bijection[A: Inhabited, B](
    f: A -> B,
    r: (A, A) -> Bool,
    op: A -> A
) {
    is_bijection_fn(f) and is_unary_congruence(r, op) implies
    is_unary_congruence(relation_pushforward(f, r), unary_op_pushforward(f, op))
} by {
    if is_bijection_fn(f) and is_unary_congruence(r, op) {
        unary_congruence_is_equivalence(r, op)
        is_equivalence(r)
        relation_pushforward_is_equivalence_of_bijection(f, r)
        is_equivalence(relation_pushforward(f, r))
        unary_congruence_respects_unary_op(r, op)
        respects_unary_op(r, op)
        relation_pushforward_respects_unary_op_of_bijection(f, r, op)
        respects_unary_op(relation_pushforward(f, r), unary_op_pushforward(f, op))
        is_unary_congruence(relation_pushforward(f, r), unary_op_pushforward(f, op))
    }
}

/// Pushforward preserves binary congruence relations along bijective transport of the operation.
theorem relation_pushforward_is_binary_congruence_of_bijection[A: Inhabited, B](
    f: A -> B,
    r: (A, A) -> Bool,
    op: (A, A) -> A
) {
    is_bijection_fn(f) and is_binary_congruence(r, op) implies
    is_binary_congruence(relation_pushforward(f, r), binary_op_pushforward(f, op))
} by {
    if is_bijection_fn(f) and is_binary_congruence(r, op) {
        binary_congruence_is_equivalence(r, op)
        is_equivalence(r)
        relation_pushforward_is_equivalence_of_bijection(f, r)
        is_equivalence(relation_pushforward(f, r))
        binary_congruence_respects_binary_op(r, op)
        respects_binary_op(r, op)
        relation_pushforward_respects_binary_op_of_bijection(f, r, op)
        respects_binary_op(relation_pushforward(f, r), binary_op_pushforward(f, op))
        is_binary_congruence(relation_pushforward(f, r), binary_op_pushforward(f, op))
    }
}

/// Bijective pushforward reflects unary compatibility for the transported operation.
theorem relation_pushforward_reflects_respects_unary_op_of_bijection[A: Inhabited, B](
    f: A -> B,
    r: (A, A) -> Bool,
    op: A -> A
) {
    is_bijection_fn(f) and respects_unary_op(relation_pushforward(f, r), unary_op_pushforward(f, op)) implies
    respects_unary_op(r, op)
} by {
    if is_bijection_fn(f) and respects_unary_op(relation_pushforward(f, r), unary_op_pushforward(f, op)) {
        bijection_fn_is_injective(f)
        is_injective_fn(f)
        forall(x: A, y: A) {
            if r(x, y) {
                relation_pushforward_intro(f, r, x, y)
                relation_pushforward(f, r, f(x), f(y))
                respects_unary_op_step(relation_pushforward(f, r), unary_op_pushforward(f, op), f(x), f(y))
                relation_pushforward(f, r, unary_op_pushforward(f, op, f(x)), unary_op_pushforward(f, op, f(y)))
                unary_op_pushforward_apply_image_of_bijection(f, op, x)
                unary_op_pushforward_apply_image_of_bijection(f, op, y)
                unary_op_pushforward(f, op, f(x)) = f(op(x))
                unary_op_pushforward(f, op, f(y)) = f(op(y))
                relation_pushforward(f, r, f(op(x)), f(op(y)))
                relation_pushforward_has_preimages(f, r, f(op(x)), f(op(y)))
                let a: A satisfy {
                    exists(b: A) {
                        f(a) = f(op(x)) and f(b) = f(op(y)) and r(a, b)
                    }
                }
                let b: A satisfy {
                    f(a) = f(op(x)) and f(b) = f(op(y)) and r(a, b)
                }
                injective_fn_eq(f, a, op(x))
                a = op(x)
                injective_fn_eq(f, b, op(y))
                b = op(y)
                r(op(x), op(y))
            }
        }
    }
}

/// Bijective pushforward reflects binary compatibility for the transported operation.
theorem relation_pushforward_reflects_respects_binary_op_of_bijection[A: Inhabited, B](
    f: A -> B,
    r: (A, A) -> Bool,
    op: (A, A) -> A
) {
    is_bijection_fn(f) and respects_binary_op(relation_pushforward(f, r), binary_op_pushforward(f, op)) implies
    respects_binary_op(r, op)
} by {
    if is_bijection_fn(f) and respects_binary_op(relation_pushforward(f, r), binary_op_pushforward(f, op)) {
        bijection_fn_is_injective(f)
        is_injective_fn(f)
        forall(x1: A, y1: A, x2: A, y2: A) {
            if r(x1, y1) and r(x2, y2) {
                relation_pushforward_intro(f, r, x1, y1)
                relation_pushforward(f, r, f(x1), f(y1))
                relation_pushforward_intro(f, r, x2, y2)
                relation_pushforward(f, r, f(x2), f(y2))
                respects_binary_op_step(
                    relation_pushforward(f, r),
                    binary_op_pushforward(f, op),
                    f(x1),
                    f(y1),
                    f(x2),
                    f(y2)
                )
                relation_pushforward(
                    f,
                    r,
                    binary_op_pushforward(f, op, f(x1), f(x2)),
                    binary_op_pushforward(f, op, f(y1), f(y2))
                )
                binary_op_pushforward_apply_image_of_bijection(f, op, x1, x2)
                binary_op_pushforward_apply_image_of_bijection(f, op, y1, y2)
                binary_op_pushforward(f, op, f(x1), f(x2)) = f(op(x1, x2))
                binary_op_pushforward(f, op, f(y1), f(y2)) = f(op(y1, y2))
                relation_pushforward(f, r, f(op(x1, x2)), f(op(y1, y2)))
                relation_pushforward_has_preimages(f, r, f(op(x1, x2)), f(op(y1, y2)))
                let a: A satisfy {
                    exists(b: A) {
                        f(a) = f(op(x1, x2)) and f(b) = f(op(y1, y2)) and r(a, b)
                    }
                }
                let b: A satisfy {
                    f(a) = f(op(x1, x2)) and f(b) = f(op(y1, y2)) and r(a, b)
                }
                injective_fn_eq(f, a, op(x1, x2))
                a = op(x1, x2)
                injective_fn_eq(f, b, op(y1, y2))
                b = op(y1, y2)
                r(op(x1, x2), op(y1, y2))
            }
        }
    }
}

/// Bijective pushforward exactly transports unary compatibility.
theorem relation_pushforward_respects_unary_op_iff_of_bijection[A: Inhabited, B](
    f: A -> B,
    r: (A, A) -> Bool,
    op: A -> A
) {
    is_bijection_fn(f) implies
    respects_unary_op(relation_pushforward(f, r), unary_op_pushforward(f, op)) = respects_unary_op(r, op)
} by {
    if is_bijection_fn(f) {
        if respects_unary_op(relation_pushforward(f, r), unary_op_pushforward(f, op)) {
            relation_pushforward_reflects_respects_unary_op_of_bijection(f, r, op)
            respects_unary_op(r, op)
        }
        if respects_unary_op(r, op) {
            relation_pushforward_respects_unary_op_of_bijection(f, r, op)
            respects_unary_op(relation_pushforward(f, r), unary_op_pushforward(f, op))
        }
        respects_unary_op(relation_pushforward(f, r), unary_op_pushforward(f, op)) = respects_unary_op(r, op)
    }
}

/// Bijective pushforward exactly transports binary compatibility.
theorem relation_pushforward_respects_binary_op_iff_of_bijection[A: Inhabited, B](
    f: A -> B,
    r: (A, A) -> Bool,
    op: (A, A) -> A
) {
    is_bijection_fn(f) implies
    respects_binary_op(relation_pushforward(f, r), binary_op_pushforward(f, op)) = respects_binary_op(r, op)
} by {
    if is_bijection_fn(f) {
        if respects_binary_op(relation_pushforward(f, r), binary_op_pushforward(f, op)) {
            relation_pushforward_reflects_respects_binary_op_of_bijection(f, r, op)
            respects_binary_op(r, op)
        }
        if respects_binary_op(r, op) {
            relation_pushforward_respects_binary_op_of_bijection(f, r, op)
            respects_binary_op(relation_pushforward(f, r), binary_op_pushforward(f, op))
        }
        respects_binary_op(relation_pushforward(f, r), binary_op_pushforward(f, op)) = respects_binary_op(r, op)
    }
}

/// Bijective pushforward reflects unary congruence relations for the transported operation.
theorem relation_pushforward_reflects_unary_congruence_of_bijection[A: Inhabited, B](
    f: A -> B,
    r: (A, A) -> Bool,
    op: A -> A
) {
    is_bijection_fn(f) and is_unary_congruence(relation_pushforward(f, r), unary_op_pushforward(f, op)) implies
    is_unary_congruence(r, op)
} by {
    if is_bijection_fn(f) and is_unary_congruence(relation_pushforward(f, r), unary_op_pushforward(f, op)) {
        relation_pullback_pushforward_of_injective(f, r)
        bijection_fn_is_injective(f)
        is_injective_fn(f)
        relation_pullback_pushforward_of_injective(f, r)
        relation_pullback(f, relation_pushforward(f, r)) = r
        unary_congruence_is_equivalence(relation_pushforward(f, r), unary_op_pushforward(f, op))
        is_equivalence(relation_pushforward(f, r))
        relation_pullback_is_equivalence(f, relation_pushforward(f, r))
        is_equivalence(relation_pullback(f, relation_pushforward(f, r)))
        is_equivalence(r)
        unary_congruence_respects_unary_op(relation_pushforward(f, r), unary_op_pushforward(f, op))
        respects_unary_op(relation_pushforward(f, r), unary_op_pushforward(f, op))
        relation_pushforward_reflects_respects_unary_op_of_bijection(f, r, op)
        respects_unary_op(r, op)
        is_unary_congruence(r, op)
    }
}

/// Bijective pushforward reflects binary congruence relations for the transported operation.
theorem relation_pushforward_reflects_binary_congruence_of_bijection[A: Inhabited, B](
    f: A -> B,
    r: (A, A) -> Bool,
    op: (A, A) -> A
) {
    is_bijection_fn(f) and is_binary_congruence(relation_pushforward(f, r), binary_op_pushforward(f, op)) implies
    is_binary_congruence(r, op)
} by {
    if is_bijection_fn(f) and is_binary_congruence(relation_pushforward(f, r), binary_op_pushforward(f, op)) {
        bijection_fn_is_injective(f)
        is_injective_fn(f)
        relation_pullback_pushforward_of_injective(f, r)
        relation_pullback(f, relation_pushforward(f, r)) = r
        binary_congruence_is_equivalence(relation_pushforward(f, r), binary_op_pushforward(f, op))
        is_equivalence(relation_pushforward(f, r))
        relation_pullback_is_equivalence(f, relation_pushforward(f, r))
        is_equivalence(relation_pullback(f, relation_pushforward(f, r)))
        is_equivalence(r)
        binary_congruence_respects_binary_op(relation_pushforward(f, r), binary_op_pushforward(f, op))
        respects_binary_op(relation_pushforward(f, r), binary_op_pushforward(f, op))
        relation_pushforward_reflects_respects_binary_op_of_bijection(f, r, op)
        respects_binary_op(r, op)
        is_binary_congruence(r, op)
    }
}

/// Bijective pushforward exactly transports unary congruence relations.
theorem relation_pushforward_is_unary_congruence_iff_of_bijection[A: Inhabited, B](
    f: A -> B,
    r: (A, A) -> Bool,
    op: A -> A
) {
    is_bijection_fn(f) implies
    is_unary_congruence(relation_pushforward(f, r), unary_op_pushforward(f, op)) = is_unary_congruence(r, op)
} by {
    if is_bijection_fn(f) {
        if is_unary_congruence(relation_pushforward(f, r), unary_op_pushforward(f, op)) {
            relation_pushforward_reflects_unary_congruence_of_bijection(f, r, op)
            is_unary_congruence(r, op)
        }
        if is_unary_congruence(r, op) {
            relation_pushforward_is_unary_congruence_of_bijection(f, r, op)
            is_unary_congruence(relation_pushforward(f, r), unary_op_pushforward(f, op))
        }
        is_unary_congruence(relation_pushforward(f, r), unary_op_pushforward(f, op)) = is_unary_congruence(r, op)
    }
}

/// Bijective pushforward exactly transports binary congruence relations.
theorem relation_pushforward_is_binary_congruence_iff_of_bijection[A: Inhabited, B](
    f: A -> B,
    r: (A, A) -> Bool,
    op: (A, A) -> A
) {
    is_bijection_fn(f) implies
    is_binary_congruence(relation_pushforward(f, r), binary_op_pushforward(f, op)) = is_binary_congruence(r, op)
} by {
    if is_bijection_fn(f) {
        if is_binary_congruence(relation_pushforward(f, r), binary_op_pushforward(f, op)) {
            relation_pushforward_reflects_binary_congruence_of_bijection(f, r, op)
            is_binary_congruence(r, op)
        }
        if is_binary_congruence(r, op) {
            relation_pushforward_is_binary_congruence_of_bijection(f, r, op)
            is_binary_congruence(relation_pushforward(f, r), binary_op_pushforward(f, op))
        }
        is_binary_congruence(relation_pushforward(f, r), binary_op_pushforward(f, op)) = is_binary_congruence(r, op)
    }
}

/// Pushing a function forward and then pulling it back along a bijection recovers the function.
theorem function_pushforward_compose_of_bijection[A: Inhabited, B, C](f: A -> B, g: A -> C) {
    is_bijection_fn(f) implies compose(function_pushforward(f, g), f) = g
} by {
    if is_bijection_fn(f) {
        forall(x: A) {
            compose(function_pushforward(f, g), f, x) = function_pushforward(f, g, f(x))
            function_pushforward_apply_image_of_bijection(f, g, x)
            function_pushforward(f, g, f(x)) = g(x)
            compose(function_pushforward(f, g), f, x) = g(x)
        }
        function_extensionality(compose(function_pushforward(f, g), f), g)
    }
}

/// Pulling a target function back and then pushing it forward agrees at each target point.
theorem function_pushforward_compose_target_of_surjective_at[A: Inhabited, B, C](f: A -> B, h: B -> C, y: B) {
    is_surjective_fn(f) implies function_pushforward(f, compose(h, f), y) = h(y)
} by {
    if is_surjective_fn(f) {
        function_pushforward(f, compose(h, f), y) = compose(h, f, inverse_fn(f, y))
        compose(h, f, inverse_fn(f, y)) = h(f(inverse_fn(f, y)))
        inverse_fn_apply_of_surjective(f, y)
        f(inverse_fn(f, y)) = y
        function_pushforward(f, compose(h, f), y) = h(y)
    }
}

/// Pulling a target function back and then pushing it forward along a surjection recovers the target function.
theorem function_pushforward_compose_target_of_surjective[A: Inhabited, B, C](f: A -> B, h: B -> C) {
    is_surjective_fn(f) implies function_pushforward(f, compose(h, f)) = h
} by {
    if is_surjective_fn(f) {
        forall(y: B) {
            function_pushforward_compose_target_of_surjective_at(f, h, y)
            function_pushforward(f, compose(h, f), y) = h(y)
        }
        function_extensionality(function_pushforward(f, compose(h, f)), h)
    }
}

/// A pushed-forward function is the unique target function whose pullback is the source function.
theorem function_pushforward_eq_of_bijection[A: Inhabited, B, C](f: A -> B, g: A -> C, h: B -> C) {
    is_bijection_fn(f) and compose(h, f) = g implies function_pushforward(f, g) = h
} by {
    if is_bijection_fn(f) and compose(h, f) = g {
        function_pushforward_compose_target_of_surjective(f, h)
        bijection_fn_is_surjective(f)
        is_surjective_fn(f)
        function_pushforward(f, compose(h, f)) = h
        function_pushforward(f, g) = h
    }
}

/// A pointwise commuting square determines the pushed-forward target function.
theorem function_pushforward_eq_of_bijection_pointwise[A: Inhabited, B, C](f: A -> B, g: A -> C, h: B -> C) {
    is_bijection_fn(f) and forall(x: A) { h(f(x)) = g(x) } implies function_pushforward(f, g) = h
} by {
    if is_bijection_fn(f) and forall(x: A) { h(f(x)) = g(x) } {
        forall(x: A) {
            compose(h, f, x) = h(f(x))
            h(f(x)) = g(x)
            compose(h, f, x) = g(x)
        }
        function_extensionality(compose(h, f), g)
        compose(h, f) = g
        function_pushforward_eq_of_bijection(f, g, h)
        function_pushforward(f, g) = h
    }
}

/// A transported unary operation agrees with any operation preserved by the bijection at each point.
theorem unary_op_pushforward_eq_of_bijection_at[A: Inhabited, B](f: A -> B, op_a: A -> A, op_b: B -> B, y: B) {
    is_bijection_fn(f) and preserves_unary_op(f, op_a, op_b) implies unary_op_pushforward(f, op_a, y) = op_b(y)
} by {
    if is_bijection_fn(f) and preserves_unary_op(f, op_a, op_b) {
        bijection_fn_is_surjective(f)
        is_surjective_fn(f)
        unary_op_pushforward(f, op_a, y) = f(op_a(inverse_fn(f, y)))
        preserves_unary_op_step(f, op_a, op_b, inverse_fn(f, y))
        f(op_a(inverse_fn(f, y))) = op_b(f(inverse_fn(f, y)))
        inverse_fn_apply_of_surjective(f, y)
        f(inverse_fn(f, y)) = y
        op_b(f(inverse_fn(f, y))) = op_b(y)
        unary_op_pushforward(f, op_a, y) = op_b(y)
    }
}

/// A transported unary operation is the unique operation making the bijection preserve it.
theorem unary_op_pushforward_eq_of_bijection[A: Inhabited, B](f: A -> B, op_a: A -> A, op_b: B -> B) {
    is_bijection_fn(f) and preserves_unary_op(f, op_a, op_b) implies unary_op_pushforward(f, op_a) = op_b
} by {
    if is_bijection_fn(f) and preserves_unary_op(f, op_a, op_b) {
        forall(y: B) {
            unary_op_pushforward_eq_of_bijection_at(f, op_a, op_b, y)
            unary_op_pushforward(f, op_a, y) = op_b(y)
        }
        function_extensionality(unary_op_pushforward(f, op_a), op_b)
    }
}

/// A transported binary operation agrees with any operation preserved by the bijection at each pair.
theorem binary_op_pushforward_eq_of_bijection_at[A: Inhabited, B](f: A -> B, op_a: (A, A) -> A, op_b: (B, B) -> B, y: B, z: B) {
    is_bijection_fn(f) and preserves_binary_op(f, op_a, op_b) implies binary_op_pushforward(f, op_a, y, z) = op_b(y, z)
} by {
    if is_bijection_fn(f) and preserves_binary_op(f, op_a, op_b) {
        bijection_fn_is_surjective(f)
        is_surjective_fn(f)
        binary_op_pushforward(f, op_a, y, z) = f(op_a(inverse_fn(f, y), inverse_fn(f, z)))
        preserves_binary_op_step(f, op_a, op_b, inverse_fn(f, y), inverse_fn(f, z))
        f(op_a(inverse_fn(f, y), inverse_fn(f, z))) = op_b(f(inverse_fn(f, y)), f(inverse_fn(f, z)))
        inverse_fn_apply_of_surjective(f, y)
        inverse_fn_apply_of_surjective(f, z)
        f(inverse_fn(f, y)) = y
        f(inverse_fn(f, z)) = z
        op_b(f(inverse_fn(f, y)), f(inverse_fn(f, z))) = op_b(y, z)
        binary_op_pushforward(f, op_a, y, z) = op_b(y, z)
    }
}

/// A transported binary operation is the unique operation making the bijection preserve it.
theorem binary_op_pushforward_eq_of_bijection[A: Inhabited, B](f: A -> B, op_a: (A, A) -> A, op_b: (B, B) -> B) {
    is_bijection_fn(f) and preserves_binary_op(f, op_a, op_b) implies binary_op_pushforward(f, op_a) = op_b
} by {
    if is_bijection_fn(f) and preserves_binary_op(f, op_a, op_b) {
        forall(y: B, z: B) {
            binary_op_pushforward_eq_of_bijection_at(f, op_a, op_b, y, z)
            binary_op_pushforward(f, op_a, y, z) = op_b(y, z)
        }
        binary_function_extensionality(binary_op_pushforward(f, op_a), op_b)
    }
}

/// Equality transport of the pushed-forward unary operation preserves unary compatibility.
theorem relation_pushforward_respects_unary_op_of_bijection_eq[A: Inhabited, B](
    f: A -> B,
    r: (A, A) -> Bool,
    op_a: A -> A,
    op_b: B -> B
) {
    is_bijection_fn(f) and preserves_unary_op(f, op_a, op_b) and respects_unary_op(r, op_a) implies
    respects_unary_op(relation_pushforward(f, r), op_b)
} by {
    if is_bijection_fn(f) and preserves_unary_op(f, op_a, op_b) and respects_unary_op(r, op_a) {
        relation_pushforward_respects_unary_op_of_bijection(f, r, op_a)
        respects_unary_op(relation_pushforward(f, r), unary_op_pushforward(f, op_a))
        unary_op_pushforward_eq_of_bijection(f, op_a, op_b)
        unary_op_pushforward(f, op_a) = op_b
        respects_unary_op_eq_op(relation_pushforward(f, r), unary_op_pushforward(f, op_a), op_b)
        respects_unary_op(relation_pushforward(f, r), op_b)
    }
}

/// Equality transport of the pushed-forward binary operation preserves binary compatibility.
theorem relation_pushforward_respects_binary_op_of_bijection_eq[A: Inhabited, B](
    f: A -> B,
    r: (A, A) -> Bool,
    op_a: (A, A) -> A,
    op_b: (B, B) -> B
) {
    is_bijection_fn(f) and preserves_binary_op(f, op_a, op_b) and respects_binary_op(r, op_a) implies
    respects_binary_op(relation_pushforward(f, r), op_b)
} by {
    if is_bijection_fn(f) and preserves_binary_op(f, op_a, op_b) and respects_binary_op(r, op_a) {
        relation_pushforward_respects_binary_op_of_bijection(f, r, op_a)
        respects_binary_op(relation_pushforward(f, r), binary_op_pushforward(f, op_a))
        binary_op_pushforward_eq_of_bijection(f, op_a, op_b)
        binary_op_pushforward(f, op_a) = op_b
        respects_binary_op_eq_op(relation_pushforward(f, r), binary_op_pushforward(f, op_a), op_b)
        respects_binary_op(relation_pushforward(f, r), op_b)
    }
}

/// Equality transport of the pushed-forward unary operation preserves unary congruence.
theorem relation_pushforward_is_unary_congruence_of_bijection_eq[A: Inhabited, B](
    f: A -> B,
    r: (A, A) -> Bool,
    op_a: A -> A,
    op_b: B -> B
) {
    is_bijection_fn(f) and preserves_unary_op(f, op_a, op_b) and is_unary_congruence(r, op_a) implies
    is_unary_congruence(relation_pushforward(f, r), op_b)
} by {
    if is_bijection_fn(f) and preserves_unary_op(f, op_a, op_b) and is_unary_congruence(r, op_a) {
        relation_pushforward_is_unary_congruence_of_bijection(f, r, op_a)
        is_unary_congruence(relation_pushforward(f, r), unary_op_pushforward(f, op_a))
        unary_op_pushforward_eq_of_bijection(f, op_a, op_b)
        unary_op_pushforward(f, op_a) = op_b
        unary_congruence_eq_op(relation_pushforward(f, r), unary_op_pushforward(f, op_a), op_b)
        is_unary_congruence(relation_pushforward(f, r), op_b)
    }
}

/// Equality transport of the pushed-forward binary operation preserves binary congruence.
theorem relation_pushforward_is_binary_congruence_of_bijection_eq[A: Inhabited, B](
    f: A -> B,
    r: (A, A) -> Bool,
    op_a: (A, A) -> A,
    op_b: (B, B) -> B
) {
    is_bijection_fn(f) and preserves_binary_op(f, op_a, op_b) and is_binary_congruence(r, op_a) implies
    is_binary_congruence(relation_pushforward(f, r), op_b)
} by {
    if is_bijection_fn(f) and preserves_binary_op(f, op_a, op_b) and is_binary_congruence(r, op_a) {
        relation_pushforward_is_binary_congruence_of_bijection(f, r, op_a)
        is_binary_congruence(relation_pushforward(f, r), binary_op_pushforward(f, op_a))
        binary_op_pushforward_eq_of_bijection(f, op_a, op_b)
        binary_op_pushforward(f, op_a) = op_b
        binary_congruence_eq_op(relation_pushforward(f, r), binary_op_pushforward(f, op_a), op_b)
        is_binary_congruence(relation_pushforward(f, r), op_b)
    }
}

/// Preserving a unary operation across a bijection is equivalent to being the pushed-forward operation.
theorem preserves_unary_op_iff_unary_op_pushforward_eq_of_bijection[A: Inhabited, B](
    f: A -> B,
    op_a: A -> A,
    op_b: B -> B
) {
    is_bijection_fn(f) implies preserves_unary_op(f, op_a, op_b) = (op_b = unary_op_pushforward(f, op_a))
} by {
    if is_bijection_fn(f) {
        if preserves_unary_op(f, op_a, op_b) {
            unary_op_pushforward_eq_of_bijection(f, op_a, op_b)
            unary_op_pushforward(f, op_a) = op_b
            op_b = unary_op_pushforward(f, op_a)
        }
        if op_b = unary_op_pushforward(f, op_a) {
            bijection_preserves_unary_op_pushforward(f, op_a)
            preserves_unary_op(f, op_a, unary_op_pushforward(f, op_a))
            preserves_unary_op_eq_codomain_op(f, op_a, unary_op_pushforward(f, op_a), op_b)
            preserves_unary_op(f, op_a, op_b)
        }
        preserves_unary_op(f, op_a, op_b) = (op_b = unary_op_pushforward(f, op_a))
    }
}

/// Preserving a binary operation across a bijection is equivalent to being the pushed-forward operation.
theorem preserves_binary_op_iff_binary_op_pushforward_eq_of_bijection[A: Inhabited, B](
    f: A -> B,
    op_a: (A, A) -> A,
    op_b: (B, B) -> B
) {
    is_bijection_fn(f) implies preserves_binary_op(f, op_a, op_b) = (op_b = binary_op_pushforward(f, op_a))
} by {
    if is_bijection_fn(f) {
        if preserves_binary_op(f, op_a, op_b) {
            binary_op_pushforward_eq_of_bijection(f, op_a, op_b)
            binary_op_pushforward(f, op_a) = op_b
            op_b = binary_op_pushforward(f, op_a)
        }
        if op_b = binary_op_pushforward(f, op_a) {
            bijection_preserves_binary_op_pushforward(f, op_a)
            preserves_binary_op(f, op_a, binary_op_pushforward(f, op_a))
            preserves_binary_op_eq_codomain_op(f, op_a, binary_op_pushforward(f, op_a), op_b)
            preserves_binary_op(f, op_a, op_b)
        }
        preserves_binary_op(f, op_a, op_b) = (op_b = binary_op_pushforward(f, op_a))
    }
}

/// Bijective pushforward exactly transports unary compatibility for any preserved target operation.
theorem relation_pushforward_respects_unary_op_iff_of_bijection_eq[A: Inhabited, B](
    f: A -> B,
    r: (A, A) -> Bool,
    op_a: A -> A,
    op_b: B -> B
) {
    is_bijection_fn(f) and preserves_unary_op(f, op_a, op_b) implies
    respects_unary_op(relation_pushforward(f, r), op_b) = respects_unary_op(r, op_a)
} by {
    if is_bijection_fn(f) and preserves_unary_op(f, op_a, op_b) {
        unary_op_pushforward_eq_of_bijection(f, op_a, op_b)
        unary_op_pushforward(f, op_a) = op_b
        relation_pushforward_respects_unary_op_iff_of_bijection(f, r, op_a)
        respects_unary_op(relation_pushforward(f, r), unary_op_pushforward(f, op_a)) = respects_unary_op(r, op_a)
        respects_unary_op(relation_pushforward(f, r), op_b) = respects_unary_op(r, op_a)
    }
}

/// Bijective pushforward exactly transports binary compatibility for any preserved target operation.
theorem relation_pushforward_respects_binary_op_iff_of_bijection_eq[A: Inhabited, B](
    f: A -> B,
    r: (A, A) -> Bool,
    op_a: (A, A) -> A,
    op_b: (B, B) -> B
) {
    is_bijection_fn(f) and preserves_binary_op(f, op_a, op_b) implies
    respects_binary_op(relation_pushforward(f, r), op_b) = respects_binary_op(r, op_a)
} by {
    if is_bijection_fn(f) and preserves_binary_op(f, op_a, op_b) {
        binary_op_pushforward_eq_of_bijection(f, op_a, op_b)
        binary_op_pushforward(f, op_a) = op_b
        relation_pushforward_respects_binary_op_iff_of_bijection(f, r, op_a)
        respects_binary_op(relation_pushforward(f, r), binary_op_pushforward(f, op_a)) = respects_binary_op(r, op_a)
        respects_binary_op(relation_pushforward(f, r), op_b) = respects_binary_op(r, op_a)
    }
}

/// Bijective pushforward exactly transports unary congruence for any preserved target operation.
theorem relation_pushforward_is_unary_congruence_iff_of_bijection_eq[A: Inhabited, B](
    f: A -> B,
    r: (A, A) -> Bool,
    op_a: A -> A,
    op_b: B -> B
) {
    is_bijection_fn(f) and preserves_unary_op(f, op_a, op_b) implies
    is_unary_congruence(relation_pushforward(f, r), op_b) = is_unary_congruence(r, op_a)
} by {
    if is_bijection_fn(f) and preserves_unary_op(f, op_a, op_b) {
        unary_op_pushforward_eq_of_bijection(f, op_a, op_b)
        unary_op_pushforward(f, op_a) = op_b
        relation_pushforward_is_unary_congruence_iff_of_bijection(f, r, op_a)
        is_unary_congruence(relation_pushforward(f, r), unary_op_pushforward(f, op_a)) = is_unary_congruence(r, op_a)
        is_unary_congruence(relation_pushforward(f, r), op_b) = is_unary_congruence(r, op_a)
    }
}

/// Bijective pushforward exactly transports binary congruence for any preserved target operation.
theorem relation_pushforward_is_binary_congruence_iff_of_bijection_eq[A: Inhabited, B](
    f: A -> B,
    r: (A, A) -> Bool,
    op_a: (A, A) -> A,
    op_b: (B, B) -> B
) {
    is_bijection_fn(f) and preserves_binary_op(f, op_a, op_b) implies
    is_binary_congruence(relation_pushforward(f, r), op_b) = is_binary_congruence(r, op_a)
} by {
    if is_bijection_fn(f) and preserves_binary_op(f, op_a, op_b) {
        binary_op_pushforward_eq_of_bijection(f, op_a, op_b)
        binary_op_pushforward(f, op_a) = op_b
        relation_pushforward_is_binary_congruence_iff_of_bijection(f, r, op_a)
        is_binary_congruence(relation_pushforward(f, r), binary_op_pushforward(f, op_a)) = is_binary_congruence(r, op_a)
        is_binary_congruence(relation_pushforward(f, r), op_b) = is_binary_congruence(r, op_a)
    }
}

/// True if a relation is compatible with the addition operation.
define respects_add[T: Add](r: (T, T) -> Bool) -> Bool {
    respects_binary_op(r, T.add)
}

/// True if a relation is an equivalence relation compatible with addition.
define is_add_congruence[T: Add](r: (T, T) -> Bool) -> Bool {
    is_binary_congruence(r, T.add)
}

/// True if a function preserves addition.
define preserves_add[A: Add, B: Add](f: A -> B) -> Bool {
    preserves_binary_op(f, A.add, B.add)
}

/// Additive compatibility is binary-operation compatibility for addition.
theorem respects_add_eq_respects_binary_op[T: Add](r: (T, T) -> Bool) {
    respects_add(r) = respects_binary_op(r, T.add)
}

/// Additive congruence is binary congruence for addition.
theorem add_congruence_eq_binary_congruence[T: Add](r: (T, T) -> Bool) {
    is_add_congruence(r) = is_binary_congruence(r, T.add)
}

/// Additive preservation is binary-operation preservation for addition.
theorem preserves_add_eq_preserves_binary_op[A: Add, B: Add](f: A -> B) {
    preserves_add(f) = preserves_binary_op(f, A.add, B.add)
}

/// An additive-compatible relation may be applied to related summands.
theorem respects_add_step[T: Add](r: (T, T) -> Bool, x1: T, y1: T, x2: T, y2: T) {
    respects_add(r) and r(x1, y1) and r(x2, y2) implies r(x1 + x2, y1 + y2)
} by {
    if respects_add(r) and r(x1, y1) and r(x2, y2) {
        respects_add(r) = respects_binary_op(r, T.add)
        respects_binary_op_step(r, T.add, x1, y1, x2, y2)
        r(T.add(x1, x2), T.add(y1, y2))
        r(x1 + x2, y1 + y2)
    }
}

/// An additive congruence may be applied to related summands.
theorem add_congruence_step[T: Add](r: (T, T) -> Bool, x1: T, y1: T, x2: T, y2: T) {
    is_add_congruence(r) and r(x1, y1) and r(x2, y2) implies r(x1 + x2, y1 + y2)
} by {
    if is_add_congruence(r) and r(x1, y1) and r(x2, y2) {
        is_add_congruence(r) = is_binary_congruence(r, T.add)
        binary_congruence_step(r, T.add, x1, y1, x2, y2)
        r(T.add(x1, x2), T.add(y1, y2))
        r(x1 + x2, y1 + y2)
    }
}

/// An additive congruence relation is an equivalence relation.
theorem add_congruence_is_equivalence[T: Add](r: (T, T) -> Bool) {
    is_add_congruence(r) implies is_equivalence(r)
} by {
    if is_add_congruence(r) {
        is_add_congruence(r) = is_binary_congruence(r, T.add)
        binary_congruence_is_equivalence(r, T.add)
        is_equivalence(r)
    }
}

/// An additive congruence relation is compatible with addition.
theorem add_congruence_respects_add[T: Add](r: (T, T) -> Bool) {
    is_add_congruence(r) implies respects_add(r)
} by {
    if is_add_congruence(r) {
        is_add_congruence(r) = is_binary_congruence(r, T.add)
        binary_congruence_respects_binary_op(r, T.add)
        respects_binary_op(r, T.add)
        respects_add(r)
    }
}

/// Equality is compatible with addition.
theorem eq_relation_respects_add[T: Add] {
    respects_add(eq_relation[T])
} by {
    eq_relation_respects_binary_op(T.add)
    respects_binary_op(eq_relation[T], T.add)
    respects_add(eq_relation[T])
}

/// Equality is an additive congruence relation.
theorem eq_relation_is_add_congruence[T: Add] {
    is_add_congruence(eq_relation[T])
} by {
    eq_relation_is_binary_congruence(T.add)
    is_binary_congruence(eq_relation[T], T.add)
    is_add_congruence(eq_relation[T])
}

/// An addition-preserving map may be rewritten at any pair of inputs.
theorem preserves_add_step[A: Add, B: Add](f: A -> B, x: A, y: A) {
    preserves_add(f) implies f(x + y) = f(x) + f(y)
} by {
    if preserves_add(f) {
        preserves_add(f) = preserves_binary_op(f, A.add, B.add)
        preserves_binary_op_step(f, A.add, B.add, x, y)
        f(A.add(x, y)) = B.add(f(x), f(y))
        f(x + y) = f(x) + f(y)
    }
}

/// Pullback preserves additive compatibility along addition-preserving maps.
theorem relation_pullback_respects_add_of_preserves_add[A: Add, B: Add](f: A -> B, r: (B, B) -> Bool) {
    respects_add(r) and preserves_add(f) implies respects_add(relation_pullback(f, r))
} by {
    if respects_add(r) and preserves_add(f) {
        respects_add(r) = respects_binary_op(r, B.add)
        preserves_add(f) = preserves_binary_op(f, A.add, B.add)
        relation_pullback_respects_binary_op_of_preserves_binary_op(f, r, A.add, B.add)
        respects_binary_op(relation_pullback(f, r), A.add)
        respects_add(relation_pullback(f, r))
    }
}

/// Pullback preserves additive congruence along addition-preserving maps.
theorem relation_pullback_is_add_congruence_of_preserves_add[A: Add, B: Add](f: A -> B, r: (B, B) -> Bool) {
    is_add_congruence(r) and preserves_add(f) implies is_add_congruence(relation_pullback(f, r))
} by {
    if is_add_congruence(r) and preserves_add(f) {
        is_add_congruence(r) = is_binary_congruence(r, B.add)
        preserves_add(f) = preserves_binary_op(f, A.add, B.add)
        relation_pullback_is_binary_congruence_of_preserves_binary_op(f, r, A.add, B.add)
        is_binary_congruence(relation_pullback(f, r), A.add)
        is_add_congruence(relation_pullback(f, r))
    }
}

/// Surjective pullback exactly detects additive compatibility along addition-preserving maps.
theorem relation_pullback_respects_add_iff_of_surjective[A: Add, B: Add](f: A -> B, r: (B, B) -> Bool) {
    is_surjective_fn(f) and preserves_add(f) implies
    respects_add(relation_pullback(f, r)) = respects_add(r)
} by {
    if is_surjective_fn(f) and preserves_add(f) {
        preserves_add(f) = preserves_binary_op(f, A.add, B.add)
        relation_pullback_respects_binary_op_iff_of_surjective(f, r, A.add, B.add)
        respects_binary_op(relation_pullback(f, r), A.add) = respects_binary_op(r, B.add)
        respects_add(relation_pullback(f, r)) = respects_add(r)
    }
}

/// Surjective pullback exactly detects additive congruence along addition-preserving maps.
theorem relation_pullback_is_add_congruence_iff_of_surjective[A: Add, B: Add](f: A -> B, r: (B, B) -> Bool) {
    is_surjective_fn(f) and preserves_add(f) implies
    is_add_congruence(relation_pullback(f, r)) = is_add_congruence(r)
} by {
    if is_surjective_fn(f) and preserves_add(f) {
        preserves_add(f) = preserves_binary_op(f, A.add, B.add)
        relation_pullback_is_binary_congruence_iff_of_surjective(f, r, A.add, B.add)
        is_binary_congruence(relation_pullback(f, r), A.add) = is_binary_congruence(r, B.add)
        is_add_congruence(relation_pullback(f, r)) = is_add_congruence(r)
    }
}

/// Bijective pushforward preserves additive compatibility along addition-preserving maps.
theorem relation_pushforward_respects_add_of_bijection[A: Add, B: Add](f: A -> B, r: (A, A) -> Bool) {
    is_bijection_fn(f) and preserves_add(f) and respects_add(r) implies
    respects_add(relation_pushforward(f, r))
} by {
    if is_bijection_fn(f) and preserves_add(f) and respects_add(r) {
        forall(u1: B, v1: B, u2: B, v2: B) {
            if relation_pushforward(f, r, u1, v1) and relation_pushforward(f, r, u2, v2) {
                relation_pushforward_has_preimages(f, r, u1, v1)
                let x1: A satisfy {
                    exists(y1: A) {
                        f(x1) = u1 and f(y1) = v1 and r(x1, y1)
                    }
                }
                let y1: A satisfy {
                    f(x1) = u1 and f(y1) = v1 and r(x1, y1)
                }
                relation_pushforward_has_preimages(f, r, u2, v2)
                let x2: A satisfy {
                    exists(y2: A) {
                        f(x2) = u2 and f(y2) = v2 and r(x2, y2)
                    }
                }
                let y2: A satisfy {
                    f(x2) = u2 and f(y2) = v2 and r(x2, y2)
                }
                respects_add_step(r, x1, y1, x2, y2)
                r(x1 + x2, y1 + y2)
                preserves_add_step(f, x1, x2)
                f(x1 + x2) = f(x1) + f(x2)
                preserves_add_step(f, y1, y2)
                f(y1 + y2) = f(y1) + f(y2)
                f(x1 + x2) = u1 + u2
                f(y1 + y2) = v1 + v2
                relation_pushforward_intro(f, r, x1 + x2, y1 + y2)
                relation_pushforward(f, r, u1 + u2, v1 + v2)
            }
        }
        respects_add(relation_pushforward(f, r)) =
            respects_binary_op(relation_pushforward(f, r), B.add)
        respects_binary_op(relation_pushforward(f, r), B.add) = forall(u1: B, v1: B, u2: B, v2: B) {
            relation_pushforward(f, r, u1, v1) and relation_pushforward(f, r, u2, v2) implies
            relation_pushforward(f, r, B.add(u1, u2), B.add(v1, v2))
        }
        respects_add(relation_pushforward(f, r))
    }
}

/// Bijective pushforward preserves additive congruence along addition-preserving maps.
theorem relation_pushforward_is_add_congruence_of_bijection[A: Add, B: Add](f: A -> B, r: (A, A) -> Bool) {
    is_bijection_fn(f) and preserves_add(f) and is_add_congruence(r) implies
    is_add_congruence(relation_pushforward(f, r))
} by {
    if is_bijection_fn(f) and preserves_add(f) and is_add_congruence(r) {
        is_add_congruence(r) = is_binary_congruence(r, A.add)
        binary_congruence_is_equivalence(r, A.add)
        is_equivalence(r)
        relation_pushforward_is_equivalence_of_bijection(f, r)
        is_equivalence(relation_pushforward(f, r))
        add_congruence_respects_add(r)
        respects_add(r)
        relation_pushforward_respects_add_of_bijection(f, r)
        respects_add(relation_pushforward(f, r))
        respects_add(relation_pushforward(f, r)) = respects_binary_op(relation_pushforward(f, r), B.add)
        is_add_congruence(relation_pushforward(f, r))
    }
}

/// True if a relation is compatible with the multiplication operation.
define respects_mul[T: Mul](r: (T, T) -> Bool) -> Bool {
    respects_binary_op(r, T.mul)
}

/// True if a relation is an equivalence relation compatible with multiplication.
define is_mul_congruence[T: Mul](r: (T, T) -> Bool) -> Bool {
    is_binary_congruence(r, T.mul)
}

/// True if a function preserves multiplication.
define preserves_mul[A: Mul, B: Mul](f: A -> B) -> Bool {
    preserves_binary_op(f, A.mul, B.mul)
}

/// Multiplicative compatibility is binary-operation compatibility for multiplication.
theorem respects_mul_eq_respects_binary_op[T: Mul](r: (T, T) -> Bool) {
    respects_mul(r) = respects_binary_op(r, T.mul)
}

/// Multiplicative congruence is binary congruence for multiplication.
theorem mul_congruence_eq_binary_congruence[T: Mul](r: (T, T) -> Bool) {
    is_mul_congruence(r) = is_binary_congruence(r, T.mul)
}

/// Multiplicative preservation is binary-operation preservation for multiplication.
theorem preserves_mul_eq_preserves_binary_op[A: Mul, B: Mul](f: A -> B) {
    preserves_mul(f) = preserves_binary_op(f, A.mul, B.mul)
}

/// A multiplicative-compatible relation may be applied to related factors.
theorem respects_mul_step[T: Mul](r: (T, T) -> Bool, x1: T, y1: T, x2: T, y2: T) {
    respects_mul(r) and r(x1, y1) and r(x2, y2) implies r(x1 * x2, y1 * y2)
} by {
    if respects_mul(r) and r(x1, y1) and r(x2, y2) {
        respects_mul(r) = respects_binary_op(r, T.mul)
        respects_binary_op_step(r, T.mul, x1, y1, x2, y2)
        r(T.mul(x1, x2), T.mul(y1, y2))
        r(x1 * x2, y1 * y2)
    }
}

/// A multiplicative congruence may be applied to related factors.
theorem mul_congruence_step[T: Mul](r: (T, T) -> Bool, x1: T, y1: T, x2: T, y2: T) {
    is_mul_congruence(r) and r(x1, y1) and r(x2, y2) implies r(x1 * x2, y1 * y2)
} by {
    if is_mul_congruence(r) and r(x1, y1) and r(x2, y2) {
        is_mul_congruence(r) = is_binary_congruence(r, T.mul)
        binary_congruence_step(r, T.mul, x1, y1, x2, y2)
        r(T.mul(x1, x2), T.mul(y1, y2))
        r(x1 * x2, y1 * y2)
    }
}

/// A multiplicative congruence relation is an equivalence relation.
theorem mul_congruence_is_equivalence[T: Mul](r: (T, T) -> Bool) {
    is_mul_congruence(r) implies is_equivalence(r)
} by {
    if is_mul_congruence(r) {
        is_mul_congruence(r) = is_binary_congruence(r, T.mul)
        binary_congruence_is_equivalence(r, T.mul)
        is_equivalence(r)
    }
}

/// A multiplicative congruence relation is compatible with multiplication.
theorem mul_congruence_respects_mul[T: Mul](r: (T, T) -> Bool) {
    is_mul_congruence(r) implies respects_mul(r)
} by {
    if is_mul_congruence(r) {
        is_mul_congruence(r) = is_binary_congruence(r, T.mul)
        binary_congruence_respects_binary_op(r, T.mul)
        respects_binary_op(r, T.mul)
        respects_mul(r)
    }
}

/// Equality is compatible with multiplication.
theorem eq_relation_respects_mul[T: Mul] {
    respects_mul(eq_relation[T])
} by {
    eq_relation_respects_binary_op(T.mul)
    respects_binary_op(eq_relation[T], T.mul)
    respects_mul(eq_relation[T])
}

/// Equality is a multiplicative congruence relation.
theorem eq_relation_is_mul_congruence[T: Mul] {
    is_mul_congruence(eq_relation[T])
} by {
    eq_relation_is_binary_congruence(T.mul)
    is_binary_congruence(eq_relation[T], T.mul)
    is_mul_congruence(eq_relation[T])
}

/// A multiplication-preserving map may be rewritten at any pair of inputs.
theorem preserves_mul_step[A: Mul, B: Mul](f: A -> B, x: A, y: A) {
    preserves_mul(f) implies f(x * y) = f(x) * f(y)
} by {
    if preserves_mul(f) {
        preserves_mul(f) = preserves_binary_op(f, A.mul, B.mul)
        preserves_binary_op_step(f, A.mul, B.mul, x, y)
        f(A.mul(x, y)) = B.mul(f(x), f(y))
        f(x * y) = f(x) * f(y)
    }
}

/// Pullback preserves multiplicative compatibility along multiplication-preserving maps.
theorem relation_pullback_respects_mul_of_preserves_mul[A: Mul, B: Mul](f: A -> B, r: (B, B) -> Bool) {
    respects_mul(r) and preserves_mul(f) implies respects_mul(relation_pullback(f, r))
} by {
    if respects_mul(r) and preserves_mul(f) {
        respects_mul(r) = respects_binary_op(r, B.mul)
        preserves_mul(f) = preserves_binary_op(f, A.mul, B.mul)
        relation_pullback_respects_binary_op_of_preserves_binary_op(f, r, A.mul, B.mul)
        respects_binary_op(relation_pullback(f, r), A.mul)
        respects_mul(relation_pullback(f, r))
    }
}

/// Pullback preserves multiplicative congruence along multiplication-preserving maps.
theorem relation_pullback_is_mul_congruence_of_preserves_mul[A: Mul, B: Mul](f: A -> B, r: (B, B) -> Bool) {
    is_mul_congruence(r) and preserves_mul(f) implies is_mul_congruence(relation_pullback(f, r))
} by {
    if is_mul_congruence(r) and preserves_mul(f) {
        is_mul_congruence(r) = is_binary_congruence(r, B.mul)
        preserves_mul(f) = preserves_binary_op(f, A.mul, B.mul)
        relation_pullback_is_binary_congruence_of_preserves_binary_op(f, r, A.mul, B.mul)
        is_binary_congruence(relation_pullback(f, r), A.mul)
        is_mul_congruence(relation_pullback(f, r))
    }
}

/// Surjective pullback exactly detects multiplicative compatibility along multiplication-preserving maps.
theorem relation_pullback_respects_mul_iff_of_surjective[A: Mul, B: Mul](f: A -> B, r: (B, B) -> Bool) {
    is_surjective_fn(f) and preserves_mul(f) implies
    respects_mul(relation_pullback(f, r)) = respects_mul(r)
} by {
    if is_surjective_fn(f) and preserves_mul(f) {
        preserves_mul(f) = preserves_binary_op(f, A.mul, B.mul)
        relation_pullback_respects_binary_op_iff_of_surjective(f, r, A.mul, B.mul)
        respects_binary_op(relation_pullback(f, r), A.mul) = respects_binary_op(r, B.mul)
        respects_mul(relation_pullback(f, r)) = respects_mul(r)
    }
}

/// Surjective pullback exactly detects multiplicative congruence along multiplication-preserving maps.
theorem relation_pullback_is_mul_congruence_iff_of_surjective[A: Mul, B: Mul](f: A -> B, r: (B, B) -> Bool) {
    is_surjective_fn(f) and preserves_mul(f) implies
    is_mul_congruence(relation_pullback(f, r)) = is_mul_congruence(r)
} by {
    if is_surjective_fn(f) and preserves_mul(f) {
        preserves_mul(f) = preserves_binary_op(f, A.mul, B.mul)
        relation_pullback_is_binary_congruence_iff_of_surjective(f, r, A.mul, B.mul)
        is_binary_congruence(relation_pullback(f, r), A.mul) = is_binary_congruence(r, B.mul)
        is_mul_congruence(relation_pullback(f, r)) = is_mul_congruence(r)
    }
}

/// Bijective pushforward preserves multiplicative compatibility along multiplication-preserving maps.
theorem relation_pushforward_respects_mul_of_bijection[A: Mul, B: Mul](f: A -> B, r: (A, A) -> Bool) {
    is_bijection_fn(f) and preserves_mul(f) and respects_mul(r) implies
    respects_mul(relation_pushforward(f, r))
} by {
    if is_bijection_fn(f) and preserves_mul(f) and respects_mul(r) {
        forall(u1: B, v1: B, u2: B, v2: B) {
            if relation_pushforward(f, r, u1, v1) and relation_pushforward(f, r, u2, v2) {
                relation_pushforward_has_preimages(f, r, u1, v1)
                let x1: A satisfy {
                    exists(y1: A) {
                        f(x1) = u1 and f(y1) = v1 and r(x1, y1)
                    }
                }
                let y1: A satisfy {
                    f(x1) = u1 and f(y1) = v1 and r(x1, y1)
                }
                relation_pushforward_has_preimages(f, r, u2, v2)
                let x2: A satisfy {
                    exists(y2: A) {
                        f(x2) = u2 and f(y2) = v2 and r(x2, y2)
                    }
                }
                let y2: A satisfy {
                    f(x2) = u2 and f(y2) = v2 and r(x2, y2)
                }
                respects_mul_step(r, x1, y1, x2, y2)
                r(x1 * x2, y1 * y2)
                preserves_mul_step(f, x1, x2)
                f(x1 * x2) = f(x1) * f(x2)
                preserves_mul_step(f, y1, y2)
                f(y1 * y2) = f(y1) * f(y2)
                f(x1 * x2) = u1 * u2
                f(y1 * y2) = v1 * v2
                relation_pushforward_intro(f, r, x1 * x2, y1 * y2)
                relation_pushforward(f, r, u1 * u2, v1 * v2)
            }
        }
        respects_mul(relation_pushforward(f, r)) =
            respects_binary_op(relation_pushforward(f, r), B.mul)
        respects_binary_op(relation_pushforward(f, r), B.mul) = forall(u1: B, v1: B, u2: B, v2: B) {
            relation_pushforward(f, r, u1, v1) and relation_pushforward(f, r, u2, v2) implies
            relation_pushforward(f, r, B.mul(u1, u2), B.mul(v1, v2))
        }
        respects_mul(relation_pushforward(f, r))
    }
}

/// Bijective pushforward preserves multiplicative congruence along multiplication-preserving maps.
theorem relation_pushforward_is_mul_congruence_of_bijection[A: Mul, B: Mul](f: A -> B, r: (A, A) -> Bool) {
    is_bijection_fn(f) and preserves_mul(f) and is_mul_congruence(r) implies
    is_mul_congruence(relation_pushforward(f, r))
} by {
    if is_bijection_fn(f) and preserves_mul(f) and is_mul_congruence(r) {
        is_mul_congruence(r) = is_binary_congruence(r, A.mul)
        binary_congruence_is_equivalence(r, A.mul)
        is_equivalence(r)
        relation_pushforward_is_equivalence_of_bijection(f, r)
        is_equivalence(relation_pushforward(f, r))
        mul_congruence_respects_mul(r)
        respects_mul(r)
        relation_pushforward_respects_mul_of_bijection(f, r)
        respects_mul(relation_pushforward(f, r))
        respects_mul(relation_pushforward(f, r)) = respects_binary_op(relation_pushforward(f, r), B.mul)
        is_mul_congruence(relation_pushforward(f, r))
    }
}

/// True if a function preserves the less-than-or-equal relation.
define preserves_lte[A: LTE, B: LTE](f: A -> B) -> Bool {
    respects_equivalence(f, A.lte, B.lte)
}

/// Order preservation is relation preservation for the less-than-or-equal relation.
theorem preserves_lte_eq_respects_equivalence[A: LTE, B: LTE](f: A -> B) {
    preserves_lte(f) = respects_equivalence(f, A.lte, B.lte)
}

/// An order-preserving map may be applied to a less-than-or-equal comparison.
theorem preserves_lte_step[A: LTE, B: LTE](f: A -> B, x: A, y: A) {
    preserves_lte(f) and x <= y implies f(x) <= f(y)
} by {
    if preserves_lte(f) and x <= y {
        preserves_lte(f) = respects_equivalence(f, A.lte, B.lte)
        respects_equivalence(f, A.lte, B.lte) = forall(a: A, b: A) {
            A.lte(a, b) implies B.lte(f(a), f(b))
        }
        f(x) <= f(y)
    }
}

/// Every function respects the pulled-back less-than-or-equal relation.
theorem function_respects_lte_pullback[A, B: LTE](f: A -> B) {
    respects_equivalence(f, relation_pullback(f, B.lte), B.lte)
} by {
    function_respects_pullback(f, B.lte)
    respects_equivalence(f, relation_pullback(f, B.lte), B.lte)
}

/// Pullback of less-than-or-equal preserves reflexive, transitive, and antisymmetric order data along injective functions.
theorem relation_pullback_lte_is_reflexive_transitive_antisymmetric_of_injective[A, B: LTE](f: A -> B) {
    is_injective_fn(f) and is_reflexive(B.lte) and is_transitive(B.lte) and is_antisymmetric(B.lte) implies
    is_reflexive(relation_pullback(f, B.lte)) and
    is_transitive(relation_pullback(f, B.lte)) and
    is_antisymmetric(relation_pullback(f, B.lte))
} by {
    if is_injective_fn(f) and is_reflexive(B.lte) and is_transitive(B.lte) and is_antisymmetric(B.lte) {
        relation_pullback_is_reflexive_transitive_antisymmetric_of_injective(f, B.lte)
        is_reflexive(relation_pullback(f, B.lte)) and
        is_transitive(relation_pullback(f, B.lte)) and
        is_antisymmetric(relation_pullback(f, B.lte))
    }
}

/// Pullback of less-than-or-equal preserves and reflects reflexive, transitive, and antisymmetric order data along bijections.
theorem relation_pullback_lte_is_reflexive_transitive_antisymmetric_iff_of_bijection[A, B: LTE](f: A -> B) {
    is_bijection_fn(f) implies
    (is_reflexive(relation_pullback(f, B.lte)) and
     is_transitive(relation_pullback(f, B.lte)) and
     is_antisymmetric(relation_pullback(f, B.lte))) =
    (is_reflexive(B.lte) and is_transitive(B.lte) and is_antisymmetric(B.lte))
} by {
    if is_bijection_fn(f) {
        relation_pullback_is_reflexive_transitive_antisymmetric_iff_of_bijection(f, B.lte)
        (is_reflexive(relation_pullback(f, B.lte)) and
         is_transitive(relation_pullback(f, B.lte)) and
         is_antisymmetric(relation_pullback(f, B.lte))) =
        (is_reflexive(B.lte) and is_transitive(B.lte) and is_antisymmetric(B.lte))
    }
}

/// Pushforward of less-than-or-equal preserves reflexive, transitive, and antisymmetric order data along bijections.
theorem relation_pushforward_lte_is_reflexive_transitive_antisymmetric_of_bijection[A: LTE, B](f: A -> B) {
    is_bijection_fn(f) and is_reflexive(A.lte) and is_transitive(A.lte) and is_antisymmetric(A.lte) implies
    is_reflexive(relation_pushforward(f, A.lte)) and
    is_transitive(relation_pushforward(f, A.lte)) and
    is_antisymmetric(relation_pushforward(f, A.lte))
} by {
    if is_bijection_fn(f) and is_reflexive(A.lte) and is_transitive(A.lte) and is_antisymmetric(A.lte) {
        relation_pushforward_is_reflexive_transitive_antisymmetric_of_bijection(f, A.lte)
        is_reflexive(relation_pushforward(f, A.lte)) and
        is_transitive(relation_pushforward(f, A.lte)) and
        is_antisymmetric(relation_pushforward(f, A.lte))
    }
}

/// Pullback of less-than-or-equal preserves order data along a bundled bijection.
theorem relation_pullback_lte_is_reflexive_transitive_antisymmetric_iff_of_bijection_map[A, B: LTE](e: Bijection[A, B]) {
    (is_reflexive(relation_pullback(e.map, B.lte)) and
     is_transitive(relation_pullback(e.map, B.lte)) and
     is_antisymmetric(relation_pullback(e.map, B.lte))) =
    (is_reflexive(B.lte) and is_transitive(B.lte) and is_antisymmetric(B.lte))
} by {
    bijection_map_is_bijection(e)
    relation_pullback_lte_is_reflexive_transitive_antisymmetric_iff_of_bijection(e.map)
}

/// Pushforward of less-than-or-equal preserves order data along a bundled bijection.
theorem relation_pushforward_lte_is_reflexive_transitive_antisymmetric_of_bijection_map[A: LTE, B](e: Bijection[A, B]) {
    is_reflexive(A.lte) and is_transitive(A.lte) and is_antisymmetric(A.lte) implies
    is_reflexive(relation_pushforward(e.map, A.lte)) and
    is_transitive(relation_pushforward(e.map, A.lte)) and
    is_antisymmetric(relation_pushforward(e.map, A.lte))
} by {
    if is_reflexive(A.lte) and is_transitive(A.lte) and is_antisymmetric(A.lte) {
        bijection_map_is_bijection(e)
        relation_pushforward_lte_is_reflexive_transitive_antisymmetric_of_bijection(e.map)
        is_reflexive(relation_pushforward(e.map, A.lte)) and
        is_transitive(relation_pushforward(e.map, A.lte)) and
        is_antisymmetric(relation_pushforward(e.map, A.lte))
    }
}
