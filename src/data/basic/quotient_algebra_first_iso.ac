/// First-isomorphism-theorem-facing characterizations of the kernel-quotient
/// induced maps, stating that each induced map separates projected
/// representatives exactly when the projections agree.

from data.basic.equivalence import kernel_relation
from algebra.add_monoid import AddMonoid, AddMonoidHom, add_monoid_hom_zero
from algebra.add_group import AddGroup, AddGroupHom, add_group_hom_zero
from algebra.monoid.monoid import Monoid, MonoidHom, monoid_hom_one
from algebra.group import Group, GroupHom, group_hom_one
from semiring import Semiring
from algebra.semiring_hom import SemiringHom, semiring_hom_zero
from algebra.ring.ring import Ring
from algebra.ring.ring_hom import RingHom, ring_hom_zero
from algebra.add_comm_group import AddCommGroup
from algebra.module.module import Module
from algebra.hom_kernel_injective import group_hom_eq_iff_inv_mul_one,
    add_group_hom_eq_iff_sub_zero, ring_hom_eq_iff_sub_zero
from data.basic.quotient_algebra import semiring_hom_kernel_quotient_induced,
    semiring_hom_kernel_quotient_project, semiring_hom_kernel_quotient_induced_injective,
    ring_hom_kernel_quotient_induced, ring_hom_kernel_quotient_project,
    ring_hom_kernel_quotient_induced_injective,
    group_hom_kernel_quotient_induced, group_hom_kernel_quotient_project,
    group_hom_kernel_quotient_induced_injective,
    monoid_hom_kernel_quotient_induced, monoid_hom_kernel_quotient_project,
    monoid_hom_kernel_quotient_induced_injective,
    add_monoid_hom_kernel_quotient_induced, add_monoid_hom_kernel_quotient_project,
    add_monoid_hom_kernel_quotient_induced_injective,
    add_group_hom_kernel_quotient_induced, add_group_hom_kernel_quotient_project,
    add_group_hom_kernel_quotient_induced_injective,
    linear_map_kernel_quotient_induced, linear_map_kernel_quotient_project,
    linear_map_kernel_quotient_induced_injective,
    semiring_hom_kernel_quotient_induced_eq_iff_image_eq,
    ring_hom_kernel_quotient_induced_eq_iff_image_eq,
    group_hom_kernel_quotient_induced_eq_iff_image_eq,
    monoid_hom_kernel_quotient_induced_eq_iff_image_eq,
    add_monoid_hom_kernel_quotient_induced_eq_iff_image_eq,
    add_group_hom_kernel_quotient_induced_eq_iff_image_eq,
    linear_map_kernel_quotient_induced_eq_iff_image_eq,
    semiring_hom_kernel_quotient_zero,
    semiring_hom_kernel_quotient_project_zero,
    ring_hom_kernel_quotient_zero, ring_hom_kernel_quotient_project_zero,
    group_hom_kernel_quotient_one, group_hom_kernel_quotient_project_one,
    monoid_hom_kernel_quotient_one, monoid_hom_kernel_quotient_project_one,
    add_monoid_hom_kernel_quotient_zero, add_monoid_hom_kernel_quotient_project_zero,
    add_group_hom_kernel_quotient_zero, add_group_hom_kernel_quotient_project_zero

/// Projections through the semiring-kernel quotient have equal induced values
/// exactly when the projections themselves are equal.
theorem semiring_hom_kernel_quotient_induced_eq_iff_project_eq[S: Semiring, T: Semiring](
    f: SemiringHom[S, T], a: S, b: S
) {
    (semiring_hom_kernel_quotient_induced(f, semiring_hom_kernel_quotient_project(f, a)) =
        semiring_hom_kernel_quotient_induced(f, semiring_hom_kernel_quotient_project(f, b))) =
        (semiring_hom_kernel_quotient_project(f, a) = semiring_hom_kernel_quotient_project(f, b))
} by {
    if semiring_hom_kernel_quotient_induced(f, semiring_hom_kernel_quotient_project(f, a)) =
        semiring_hom_kernel_quotient_induced(f, semiring_hom_kernel_quotient_project(f, b)) {
        semiring_hom_kernel_quotient_induced_injective(f, a, b)
    }
}

/// Projections through the ring-kernel quotient have equal induced values
/// exactly when the projections themselves are equal.
theorem ring_hom_kernel_quotient_induced_eq_iff_project_eq[R: Ring, S: Ring](
    f: RingHom[R, S], a: R, b: R
) {
    (ring_hom_kernel_quotient_induced(f, ring_hom_kernel_quotient_project(f, a)) =
        ring_hom_kernel_quotient_induced(f, ring_hom_kernel_quotient_project(f, b))) =
        (ring_hom_kernel_quotient_project(f, a) = ring_hom_kernel_quotient_project(f, b))
} by {
    if ring_hom_kernel_quotient_induced(f, ring_hom_kernel_quotient_project(f, a)) =
        ring_hom_kernel_quotient_induced(f, ring_hom_kernel_quotient_project(f, b)) {
        ring_hom_kernel_quotient_induced_injective(f, a, b)
    }
}

/// Projections through the group-kernel quotient have equal induced values
/// exactly when the projections themselves are equal.
theorem group_hom_kernel_quotient_induced_eq_iff_project_eq[G: Group, H: Group](
    f: GroupHom[G, H], a: G, b: G
) {
    (group_hom_kernel_quotient_induced(f, group_hom_kernel_quotient_project(f, a)) =
        group_hom_kernel_quotient_induced(f, group_hom_kernel_quotient_project(f, b))) =
        (group_hom_kernel_quotient_project(f, a) = group_hom_kernel_quotient_project(f, b))
} by {
    if group_hom_kernel_quotient_induced(f, group_hom_kernel_quotient_project(f, a)) =
        group_hom_kernel_quotient_induced(f, group_hom_kernel_quotient_project(f, b)) {
        group_hom_kernel_quotient_induced_injective(f, a, b)
    }
}

/// Projections through the monoid-kernel quotient have equal induced values
/// exactly when the projections themselves are equal.
theorem monoid_hom_kernel_quotient_induced_eq_iff_project_eq[M: Monoid, N: Monoid](
    f: MonoidHom[M, N], a: M, b: M
) {
    (monoid_hom_kernel_quotient_induced(f, monoid_hom_kernel_quotient_project(f, a)) =
        monoid_hom_kernel_quotient_induced(f, monoid_hom_kernel_quotient_project(f, b))) =
        (monoid_hom_kernel_quotient_project(f, a) = monoid_hom_kernel_quotient_project(f, b))
} by {
    if monoid_hom_kernel_quotient_induced(f, monoid_hom_kernel_quotient_project(f, a)) =
        monoid_hom_kernel_quotient_induced(f, monoid_hom_kernel_quotient_project(f, b)) {
        monoid_hom_kernel_quotient_induced_injective(f, a, b)
    }
}

/// Projections through the additive-monoid-kernel quotient have equal induced values
/// exactly when the projections themselves are equal.
theorem add_monoid_hom_kernel_quotient_induced_eq_iff_project_eq[A: AddMonoid, B: AddMonoid](
    f: AddMonoidHom[A, B], a: A, b: A
) {
    (add_monoid_hom_kernel_quotient_induced(f, add_monoid_hom_kernel_quotient_project(f, a)) =
        add_monoid_hom_kernel_quotient_induced(f, add_monoid_hom_kernel_quotient_project(f, b))) =
        (add_monoid_hom_kernel_quotient_project(f, a) = add_monoid_hom_kernel_quotient_project(f, b))
} by {
    if add_monoid_hom_kernel_quotient_induced(f, add_monoid_hom_kernel_quotient_project(f, a)) =
        add_monoid_hom_kernel_quotient_induced(f, add_monoid_hom_kernel_quotient_project(f, b)) {
        add_monoid_hom_kernel_quotient_induced_injective(f, a, b)
    }
}

/// Projections through the additive-group-kernel quotient have equal induced values
/// exactly when the projections themselves are equal.
theorem add_group_hom_kernel_quotient_induced_eq_iff_project_eq[A: AddGroup, B: AddGroup](
    f: AddGroupHom[A, B], a: A, b: A
) {
    (add_group_hom_kernel_quotient_induced(f, add_group_hom_kernel_quotient_project(f, a)) =
        add_group_hom_kernel_quotient_induced(f, add_group_hom_kernel_quotient_project(f, b))) =
        (add_group_hom_kernel_quotient_project(f, a) = add_group_hom_kernel_quotient_project(f, b))
} by {
    if add_group_hom_kernel_quotient_induced(f, add_group_hom_kernel_quotient_project(f, a)) =
        add_group_hom_kernel_quotient_induced(f, add_group_hom_kernel_quotient_project(f, b)) {
        add_group_hom_kernel_quotient_induced_injective(f, a, b)
    }
}

/// Projections through the linear-map-kernel quotient have equal induced values
/// exactly when the projections themselves are equal.
theorem linear_map_kernel_quotient_induced_eq_iff_project_eq[R: Ring, M: AddCommGroup, N: AddCommGroup](
    src: Module[R, M], dst: Module[R, N], f: M -> N, a: M, b: M
) {
    (linear_map_kernel_quotient_induced(src, dst, f,
        linear_map_kernel_quotient_project(src, dst, f, a)) =
        linear_map_kernel_quotient_induced(src, dst, f,
            linear_map_kernel_quotient_project(src, dst, f, b))) =
        (linear_map_kernel_quotient_project(src, dst, f, a) =
            linear_map_kernel_quotient_project(src, dst, f, b))
} by {
    if linear_map_kernel_quotient_induced(src, dst, f,
        linear_map_kernel_quotient_project(src, dst, f, a)) =
        linear_map_kernel_quotient_induced(src, dst, f,
            linear_map_kernel_quotient_project(src, dst, f, b)) {
        linear_map_kernel_quotient_induced_injective(src, dst, f, a, b)
    }
}

/// Two semiring representatives have equal homomorphism values exactly when
/// their kernel-quotient projections agree, the well-definedness-and-injectivity
/// core of the first isomorphism theorem.
theorem semiring_hom_kernel_quotient_image_eq_iff_project_eq[S: Semiring, T: Semiring](
    f: SemiringHom[S, T], a: S, b: S
) {
    (f.hom(a) = f.hom(b)) =
        (semiring_hom_kernel_quotient_project(f, a) = semiring_hom_kernel_quotient_project(f, b))
} by {
    semiring_hom_kernel_quotient_induced_eq_iff_image_eq(f, a, b)
    semiring_hom_kernel_quotient_induced_eq_iff_project_eq(f, a, b)
}

/// Two ring representatives have equal homomorphism values exactly when their
/// kernel-quotient projections agree.
theorem ring_hom_kernel_quotient_image_eq_iff_project_eq[R: Ring, S: Ring](
    f: RingHom[R, S], a: R, b: R
) {
    (f.hom(a) = f.hom(b)) =
        (ring_hom_kernel_quotient_project(f, a) = ring_hom_kernel_quotient_project(f, b))
} by {
    ring_hom_kernel_quotient_induced_eq_iff_image_eq(f, a, b)
    ring_hom_kernel_quotient_induced_eq_iff_project_eq(f, a, b)
}

/// Two group representatives have equal homomorphism values exactly when their
/// kernel-quotient projections agree.
theorem group_hom_kernel_quotient_image_eq_iff_project_eq[G: Group, H: Group](
    f: GroupHom[G, H], a: G, b: G
) {
    (f.hom(a) = f.hom(b)) =
        (group_hom_kernel_quotient_project(f, a) = group_hom_kernel_quotient_project(f, b))
} by {
    group_hom_kernel_quotient_induced_eq_iff_image_eq(f, a, b)
    group_hom_kernel_quotient_induced_eq_iff_project_eq(f, a, b)
}

/// Two monoid representatives have equal homomorphism values exactly when their
/// kernel-quotient projections agree.
theorem monoid_hom_kernel_quotient_image_eq_iff_project_eq[M: Monoid, N: Monoid](
    f: MonoidHom[M, N], a: M, b: M
) {
    (f.hom(a) = f.hom(b)) =
        (monoid_hom_kernel_quotient_project(f, a) = monoid_hom_kernel_quotient_project(f, b))
} by {
    monoid_hom_kernel_quotient_induced_eq_iff_image_eq(f, a, b)
    monoid_hom_kernel_quotient_induced_eq_iff_project_eq(f, a, b)
}

/// Two additive-monoid representatives have equal homomorphism values exactly
/// when their kernel-quotient projections agree.
theorem add_monoid_hom_kernel_quotient_image_eq_iff_project_eq[A: AddMonoid, B: AddMonoid](
    f: AddMonoidHom[A, B], a: A, b: A
) {
    (f.hom(a) = f.hom(b)) =
        (add_monoid_hom_kernel_quotient_project(f, a) = add_monoid_hom_kernel_quotient_project(f, b))
} by {
    add_monoid_hom_kernel_quotient_induced_eq_iff_image_eq(f, a, b)
    add_monoid_hom_kernel_quotient_induced_eq_iff_project_eq(f, a, b)
}

/// Two additive-group representatives have equal homomorphism values exactly
/// when their kernel-quotient projections agree.
theorem add_group_hom_kernel_quotient_image_eq_iff_project_eq[A: AddGroup, B: AddGroup](
    f: AddGroupHom[A, B], a: A, b: A
) {
    (f.hom(a) = f.hom(b)) =
        (add_group_hom_kernel_quotient_project(f, a) = add_group_hom_kernel_quotient_project(f, b))
} by {
    add_group_hom_kernel_quotient_induced_eq_iff_image_eq(f, a, b)
    add_group_hom_kernel_quotient_induced_eq_iff_project_eq(f, a, b)
}

/// Two module elements have equal images exactly when their linear-map
/// kernel-quotient projections agree.
theorem linear_map_kernel_quotient_image_eq_iff_project_eq[R: Ring, M: AddCommGroup, N: AddCommGroup](
    src: Module[R, M], dst: Module[R, N], f: M -> N, a: M, b: M
) {
    (f(a) = f(b)) =
        (linear_map_kernel_quotient_project(src, dst, f, a) =
            linear_map_kernel_quotient_project(src, dst, f, b))
} by {
    linear_map_kernel_quotient_induced_eq_iff_image_eq(src, dst, f, a, b)
    linear_map_kernel_quotient_induced_eq_iff_project_eq(src, dst, f, a, b)
}

/// A representative lies in the group-kernel quotient fiber over the quotient
/// identity exactly when it lies in the kernel of the homomorphism.
theorem group_hom_kernel_quotient_project_eq_one_iff_image_eq_one[G: Group, H: Group](
    f: GroupHom[G, H], a: G
) {
    (group_hom_kernel_quotient_project(f, a) = group_hom_kernel_quotient_one(f)) =
        (f.hom(a) = H.1)
} by {
    group_hom_kernel_quotient_image_eq_iff_project_eq(f, a, G.1)
    group_hom_one(f)
    group_hom_kernel_quotient_project_one(f)
}

/// A representative lies in the monoid-kernel quotient fiber over the quotient
/// identity exactly when its homomorphism value is the codomain identity.
theorem monoid_hom_kernel_quotient_project_eq_one_iff_image_eq_one[M: Monoid, N: Monoid](
    f: MonoidHom[M, N], a: M
) {
    (monoid_hom_kernel_quotient_project(f, a) = monoid_hom_kernel_quotient_one(f)) =
        (f.hom(a) = N.1)
} by {
    monoid_hom_kernel_quotient_image_eq_iff_project_eq(f, a, M.1)
    monoid_hom_one(f)
    monoid_hom_kernel_quotient_project_one(f)
}

/// A representative lies in the additive-group-kernel quotient fiber over the
/// quotient zero exactly when its homomorphism value is the codomain zero.
theorem add_group_hom_kernel_quotient_project_eq_zero_iff_image_eq_zero[A: AddGroup, B: AddGroup](
    f: AddGroupHom[A, B], a: A
) {
    (add_group_hom_kernel_quotient_project(f, a) = add_group_hom_kernel_quotient_zero(f)) =
        (f.hom(a) = B.0)
} by {
    add_group_hom_kernel_quotient_image_eq_iff_project_eq(f, a, A.0)
    add_group_hom_zero(f)
    add_group_hom_kernel_quotient_project_zero(f)
}

/// A representative lies in the additive-monoid-kernel quotient fiber over the
/// quotient zero exactly when its homomorphism value is the codomain zero.
theorem add_monoid_hom_kernel_quotient_project_eq_zero_iff_image_eq_zero[A: AddMonoid, B: AddMonoid](
    f: AddMonoidHom[A, B], a: A
) {
    (add_monoid_hom_kernel_quotient_project(f, a) = add_monoid_hom_kernel_quotient_zero(f)) =
        (f.hom(a) = B.0)
} by {
    add_monoid_hom_kernel_quotient_image_eq_iff_project_eq(f, a, A.0)
    add_monoid_hom_zero(f)
    add_monoid_hom_kernel_quotient_project_zero(f)
}

/// A representative lies in the semiring-kernel quotient fiber over the quotient
/// zero exactly when its homomorphism value is the codomain zero.
theorem semiring_hom_kernel_quotient_project_eq_zero_iff_image_eq_zero[S: Semiring, T: Semiring](
    f: SemiringHom[S, T], a: S
) {
    (semiring_hom_kernel_quotient_project(f, a) = semiring_hom_kernel_quotient_zero(f)) =
        (f.hom(a) = T.0)
} by {
    semiring_hom_kernel_quotient_image_eq_iff_project_eq(f, a, S.0)
    semiring_hom_zero(f)
    semiring_hom_kernel_quotient_project_zero(f)
}

/// A representative lies in the ring-kernel quotient fiber over the quotient
/// zero exactly when its homomorphism value is the codomain zero.
theorem ring_hom_kernel_quotient_project_eq_zero_iff_image_eq_zero[R: Ring, S: Ring](
    f: RingHom[R, S], a: R
) {
    (ring_hom_kernel_quotient_project(f, a) = ring_hom_kernel_quotient_zero(f)) =
        (f.hom(a) = S.0)
} by {
    ring_hom_kernel_quotient_image_eq_iff_project_eq(f, a, R.0)
    ring_hom_zero(f)
    ring_hom_kernel_quotient_project_zero(f)
}

/// Two group representatives have equal kernel-quotient projections exactly
/// when the homomorphism sends the product of the inverse of the first with
/// the second to the codomain identity.
theorem group_hom_kernel_quotient_project_eq_iff_inv_mul_one[G: Group, H: Group](
    f: GroupHom[G, H], a: G, b: G
) {
    (group_hom_kernel_quotient_project(f, a) = group_hom_kernel_quotient_project(f, b)) =
        (f.hom(a.inverse * b) = H.1)
} by {
    group_hom_kernel_quotient_image_eq_iff_project_eq(f, a, b)
    group_hom_eq_iff_inv_mul_one(f, a, b)
}

/// Two additive-group representatives have equal kernel-quotient projections
/// exactly when the homomorphism sends the difference of the second and the
/// first to the codomain zero.
theorem add_group_hom_kernel_quotient_project_eq_iff_sub_zero[A: AddGroup, B: AddGroup](
    f: AddGroupHom[A, B], a: A, b: A
) {
    (add_group_hom_kernel_quotient_project(f, a) = add_group_hom_kernel_quotient_project(f, b)) =
        (f.hom(b - a) = B.0)
} by {
    add_group_hom_kernel_quotient_image_eq_iff_project_eq(f, a, b)
    add_group_hom_eq_iff_sub_zero(f, a, b)
}

/// Two ring representatives have equal kernel-quotient projections exactly when
/// the homomorphism sends the difference of the second and the first to the
/// codomain zero.
theorem ring_hom_kernel_quotient_project_eq_iff_sub_zero[R: Ring, S: Ring](
    f: RingHom[R, S], a: R, b: R
) {
    (ring_hom_kernel_quotient_project(f, a) = ring_hom_kernel_quotient_project(f, b)) =
        (f.hom(b - a) = S.0)
} by {
    ring_hom_kernel_quotient_image_eq_iff_project_eq(f, a, b)
    ring_hom_eq_iff_sub_zero(f, a, b)
}

/// Two semiring representatives have equal kernel-quotient projections exactly
/// when they are related by the kernel relation of the homomorphism.
theorem semiring_hom_kernel_quotient_project_eq_iff_kernel_relation[S: Semiring, T: Semiring](
    f: SemiringHom[S, T], a: S, b: S
) {
    (semiring_hom_kernel_quotient_project(f, a) = semiring_hom_kernel_quotient_project(f, b)) =
        kernel_relation(f.hom, a, b)
} by {
    semiring_hom_kernel_quotient_image_eq_iff_project_eq(f, a, b)
    kernel_relation(f.hom, a, b) = (f.hom(a) = f.hom(b))
}

/// Two ring representatives have equal kernel-quotient projections exactly
/// when they are related by the kernel relation of the homomorphism.
theorem ring_hom_kernel_quotient_project_eq_iff_kernel_relation[R: Ring, S: Ring](
    f: RingHom[R, S], a: R, b: R
) {
    (ring_hom_kernel_quotient_project(f, a) = ring_hom_kernel_quotient_project(f, b)) =
        kernel_relation(f.hom, a, b)
} by {
    ring_hom_kernel_quotient_image_eq_iff_project_eq(f, a, b)
    kernel_relation(f.hom, a, b) = (f.hom(a) = f.hom(b))
}

/// Two group representatives have equal kernel-quotient projections exactly
/// when they are related by the kernel relation of the homomorphism.
theorem group_hom_kernel_quotient_project_eq_iff_kernel_relation[G: Group, H: Group](
    f: GroupHom[G, H], a: G, b: G
) {
    (group_hom_kernel_quotient_project(f, a) = group_hom_kernel_quotient_project(f, b)) =
        kernel_relation(f.hom, a, b)
} by {
    group_hom_kernel_quotient_image_eq_iff_project_eq(f, a, b)
    kernel_relation(f.hom, a, b) = (f.hom(a) = f.hom(b))
}

/// Two monoid representatives have equal kernel-quotient projections exactly
/// when they are related by the kernel relation of the homomorphism.
theorem monoid_hom_kernel_quotient_project_eq_iff_kernel_relation[M: Monoid, N: Monoid](
    f: MonoidHom[M, N], a: M, b: M
) {
    (monoid_hom_kernel_quotient_project(f, a) = monoid_hom_kernel_quotient_project(f, b)) =
        kernel_relation(f.hom, a, b)
} by {
    monoid_hom_kernel_quotient_image_eq_iff_project_eq(f, a, b)
    kernel_relation(f.hom, a, b) = (f.hom(a) = f.hom(b))
}

/// Two additive-monoid representatives have equal kernel-quotient projections
/// exactly when they are related by the kernel relation of the homomorphism.
theorem add_monoid_hom_kernel_quotient_project_eq_iff_kernel_relation[A: AddMonoid, B: AddMonoid](
    f: AddMonoidHom[A, B], a: A, b: A
) {
    (add_monoid_hom_kernel_quotient_project(f, a) = add_monoid_hom_kernel_quotient_project(f, b)) =
        kernel_relation(f.hom, a, b)
} by {
    add_monoid_hom_kernel_quotient_image_eq_iff_project_eq(f, a, b)
    kernel_relation(f.hom, a, b) = (f.hom(a) = f.hom(b))
}

/// Two additive-group representatives have equal kernel-quotient projections
/// exactly when they are related by the kernel relation of the homomorphism.
theorem add_group_hom_kernel_quotient_project_eq_iff_kernel_relation[A: AddGroup, B: AddGroup](
    f: AddGroupHom[A, B], a: A, b: A
) {
    (add_group_hom_kernel_quotient_project(f, a) = add_group_hom_kernel_quotient_project(f, b)) =
        kernel_relation(f.hom, a, b)
} by {
    add_group_hom_kernel_quotient_image_eq_iff_project_eq(f, a, b)
    kernel_relation(f.hom, a, b) = (f.hom(a) = f.hom(b))
}

/// Two module elements have equal linear-map kernel-quotient projections
/// exactly when they are related by the kernel relation of the underlying map.
theorem linear_map_kernel_quotient_project_eq_iff_kernel_relation[R: Ring, M: AddCommGroup, N: AddCommGroup](
    src: Module[R, M], dst: Module[R, N], f: M -> N, a: M, b: M
) {
    (linear_map_kernel_quotient_project(src, dst, f, a) =
        linear_map_kernel_quotient_project(src, dst, f, b)) =
        kernel_relation(f, a, b)
} by {
    linear_map_kernel_quotient_image_eq_iff_project_eq(src, dst, f, a, b)
    kernel_relation(f, a, b) = (f(a) = f(b))
}

/// Induced values on semiring-kernel-quotient projections agree exactly when
/// the underlying representatives are related by the kernel relation.
theorem semiring_hom_kernel_quotient_induced_eq_iff_kernel_relation[S: Semiring, T: Semiring](
    f: SemiringHom[S, T], a: S, b: S
) {
    (semiring_hom_kernel_quotient_induced(f, semiring_hom_kernel_quotient_project(f, a)) =
        semiring_hom_kernel_quotient_induced(f, semiring_hom_kernel_quotient_project(f, b))) =
        kernel_relation(f.hom, a, b)
} by {
    semiring_hom_kernel_quotient_induced_eq_iff_image_eq(f, a, b)
    kernel_relation(f.hom, a, b) = (f.hom(a) = f.hom(b))
}

/// Induced values on ring-kernel-quotient projections agree exactly when the
/// underlying representatives are related by the kernel relation.
theorem ring_hom_kernel_quotient_induced_eq_iff_kernel_relation[R: Ring, S: Ring](
    f: RingHom[R, S], a: R, b: R
) {
    (ring_hom_kernel_quotient_induced(f, ring_hom_kernel_quotient_project(f, a)) =
        ring_hom_kernel_quotient_induced(f, ring_hom_kernel_quotient_project(f, b))) =
        kernel_relation(f.hom, a, b)
} by {
    ring_hom_kernel_quotient_induced_eq_iff_image_eq(f, a, b)
    kernel_relation(f.hom, a, b) = (f.hom(a) = f.hom(b))
}

/// Induced values on group-kernel-quotient projections agree exactly when the
/// underlying representatives are related by the kernel relation.
theorem group_hom_kernel_quotient_induced_eq_iff_kernel_relation[G: Group, H: Group](
    f: GroupHom[G, H], a: G, b: G
) {
    (group_hom_kernel_quotient_induced(f, group_hom_kernel_quotient_project(f, a)) =
        group_hom_kernel_quotient_induced(f, group_hom_kernel_quotient_project(f, b))) =
        kernel_relation(f.hom, a, b)
} by {
    group_hom_kernel_quotient_induced_eq_iff_image_eq(f, a, b)
    kernel_relation(f.hom, a, b) = (f.hom(a) = f.hom(b))
}

/// Induced values on monoid-kernel-quotient projections agree exactly when the
/// underlying representatives are related by the kernel relation.
theorem monoid_hom_kernel_quotient_induced_eq_iff_kernel_relation[M: Monoid, N: Monoid](
    f: MonoidHom[M, N], a: M, b: M
) {
    (monoid_hom_kernel_quotient_induced(f, monoid_hom_kernel_quotient_project(f, a)) =
        monoid_hom_kernel_quotient_induced(f, monoid_hom_kernel_quotient_project(f, b))) =
        kernel_relation(f.hom, a, b)
} by {
    monoid_hom_kernel_quotient_induced_eq_iff_image_eq(f, a, b)
    kernel_relation(f.hom, a, b) = (f.hom(a) = f.hom(b))
}

/// Induced values on additive-monoid-kernel-quotient projections agree exactly
/// when the underlying representatives are related by the kernel relation.
theorem add_monoid_hom_kernel_quotient_induced_eq_iff_kernel_relation[A: AddMonoid, B: AddMonoid](
    f: AddMonoidHom[A, B], a: A, b: A
) {
    (add_monoid_hom_kernel_quotient_induced(f, add_monoid_hom_kernel_quotient_project(f, a)) =
        add_monoid_hom_kernel_quotient_induced(f, add_monoid_hom_kernel_quotient_project(f, b))) =
        kernel_relation(f.hom, a, b)
} by {
    add_monoid_hom_kernel_quotient_induced_eq_iff_image_eq(f, a, b)
    kernel_relation(f.hom, a, b) = (f.hom(a) = f.hom(b))
}

/// Induced values on additive-group-kernel-quotient projections agree exactly
/// when the underlying representatives are related by the kernel relation.
theorem add_group_hom_kernel_quotient_induced_eq_iff_kernel_relation[A: AddGroup, B: AddGroup](
    f: AddGroupHom[A, B], a: A, b: A
) {
    (add_group_hom_kernel_quotient_induced(f, add_group_hom_kernel_quotient_project(f, a)) =
        add_group_hom_kernel_quotient_induced(f, add_group_hom_kernel_quotient_project(f, b))) =
        kernel_relation(f.hom, a, b)
} by {
    add_group_hom_kernel_quotient_induced_eq_iff_image_eq(f, a, b)
    kernel_relation(f.hom, a, b) = (f.hom(a) = f.hom(b))
}

/// Induced values on linear-map-kernel-quotient projections agree exactly when
/// the underlying module elements are related by the kernel relation.
theorem linear_map_kernel_quotient_induced_eq_iff_kernel_relation[R: Ring, M: AddCommGroup, N: AddCommGroup](
    src: Module[R, M], dst: Module[R, N], f: M -> N, a: M, b: M
) {
    (linear_map_kernel_quotient_induced(src, dst, f, linear_map_kernel_quotient_project(src, dst, f, a)) =
        linear_map_kernel_quotient_induced(src, dst, f, linear_map_kernel_quotient_project(src, dst, f, b))) =
        kernel_relation(f, a, b)
} by {
    linear_map_kernel_quotient_induced_eq_iff_image_eq(src, dst, f, a, b)
    kernel_relation(f, a, b) = (f(a) = f(b))
}
