from data.basic.functions import Inhabited
from list import List
from data.basic.logic import exists_intro
from pair import Pair
from data.basic.set import Set

/// True if exactly one element satisfies the predicate.
define exists_unique[T](p: T -> Bool) -> Bool {
    exists(x: T) {
        p(x) and forall(y: T) {
            p(y) implies y = x
        }
    }
}

/// Unique existence gives existence.
theorem exists_unique_exists[T](p: T -> Bool) {
    exists_unique(p) implies exists(x: T) {
        p(x)
    }
} by {
    if exists_unique(p) {
        let x: T satisfy {
            p(x) and forall(y: T) {
                p(y) implies y = x
            }
        }
        exists(witness: T) {
            witness = x and p(witness)
        }
    }
}

/// Unique existence gives uniqueness of any two witnesses.
theorem exists_unique_eq[T](p: T -> Bool, x: T, y: T) {
    exists_unique(p) and p(x) and p(y) implies x = y
} by {
    if exists_unique(p) and p(x) and p(y) {
        let witness: T satisfy {
            p(witness) and forall(z: T) {
                p(z) implies z = witness
            }
        }
        x = witness
        y = witness
    }
}

/// Existence together with uniqueness gives unique existence.
theorem exists_unique_intro[T](p: T -> Bool, x: T) {
    p(x) and forall(y: T) {
        p(y) implies y = x
    } implies exists_unique(p)
} by {
    if p(x) and forall(y: T) { p(y) implies y = x } {
        exists(witness: T) {
            witness = x and p(witness) and forall(y: T) {
                p(y) implies y = witness
            }
        }
    }
}

/// Existence together with pairwise uniqueness gives unique existence.
theorem exists_unique_of_exists_and_pairwise[T](p: T -> Bool) {
    exists(x: T) {
        p(x)
    } and forall(x: T, y: T) {
        p(x) and p(y) implies x = y
    } implies exists_unique(p)
} by {
    if exists(x: T) { p(x) } and forall(x: T, y: T) { p(x) and p(y) implies x = y } {
        let witness: T satisfy {
            p(witness)
        }
        forall(y: T) {
            if p(y) {
                y = witness
            }
        }
        exists_unique_intro(p, witness)
        exists_unique(p)
    }
}

/// Equivalent predicates have the same unique-existence property.
theorem exists_unique_congr[T](p: T -> Bool, q: T -> Bool) {
    (forall(x: T) { p(x) = q(x) }) implies
    (exists_unique(p) = exists_unique(q))
} by {
    if forall(x: T) { p(x) = q(x) } {
        if exists_unique(p) {
            let x: T satisfy {
                p(x) and forall(y: T) {
                    p(y) implies y = x
                }
            }
            q(x)
            forall(y: T) {
                if q(y) {
                    p(y)
                    y = x
                }
            }
            exists_unique(q)
        }
        if exists_unique(q) {
            let x: T satisfy {
                q(x) and forall(y: T) {
                    q(y) implies y = x
                }
            }
            p(x)
            forall(y: T) {
                if p(y) {
                    q(y)
                    y = x
                }
            }
            exists_unique(p)
        }
    }
}

/// A chosen witness for a satisfiable predicate on an inhabited type.
/// Use this when the witness will be reused by later statements; use `satisfy`
/// directly inside a proof when the witness is only needed locally.
let choose_of_exists[T: Inhabited](p: T -> Bool) -> result: T satisfy {
    exists(x: T) {
        p(x)
    } implies p(result)
} by {
    if exists(x: T) { p(x) } {
        let x: T satisfy {
            p(x)
        }
    } else {
        let x: T satisfy {
            true
        }
    }
}

/// A chosen witness when one exists, and otherwise a specified default element.
/// Use this when a total construction needs a specified value in the empty case.
let choose_or_default[T](p: T -> Bool, default: T) -> result: T satisfy {
    (exists(x: T) {
        p(x)
    } implies p(result)) and
    (not exists(x: T) {
        p(x)
    } implies result = default)
} by {
    if exists(x: T) { p(x) } {
        let x: T satisfy {
            p(x)
        }
    } else {
        let x: T = default
    }
}

/// The chosen witness satisfies the predicate when a witness exists.
theorem choose_of_exists_spec[T: Inhabited](p: T -> Bool) {
    exists(x: T) {
        p(x)
    } implies p(choose_of_exists(p))
}

/// A chosen witness for a satisfiable predicate on an inhabited type.
/// This is the preferred name for new developments.
let choose_witness[T: Inhabited](p: T -> Bool) -> result: T satisfy {
    exists(x: T) {
        p(x)
    } implies p(result)
} by {
    let x: T = choose_of_exists(p)
    if exists(y: T) { p(y) } {
        choose_of_exists_spec(p)
    }
}

/// The chosen witness satisfies the predicate when a witness exists.
theorem choose_witness_spec[T: Inhabited](p: T -> Bool) {
    exists(x: T) {
        p(x)
    } implies p(choose_witness(p))
}

/// A chosen witness satisfies a uniquely satisfiable predicate.
theorem choose_witness_unique_spec[T: Inhabited](p: T -> Bool) {
    exists_unique(p) implies p(choose_witness(p))
} by {
    if exists_unique(p) {
        exists_unique_exists(p)
        exists(x: T) {
            p(x)
        }
        choose_witness_spec(p)
    }
}

/// The preferred chosen witness is equal to any witness of a uniquely satisfiable predicate.
theorem choose_witness_unique_eq[T: Inhabited](p: T -> Bool, x: T) {
    exists_unique(p) and p(x) implies choose_witness(p) = x
} by {
    if exists_unique(p) and p(x) {
        choose_witness_unique_spec(p)
        p(choose_witness(p))
        exists_unique_eq(p, choose_witness(p), x)
    }
}

/// Equivalent uniquely satisfiable predicates have the same preferred chosen witness.
theorem choose_witness_unique_congr[T: Inhabited](p: T -> Bool, q: T -> Bool) {
    exists_unique(p) and forall(x: T) {
        p(x) = q(x)
    } implies choose_witness(p) = choose_witness(q)
} by {
    if exists_unique(p) and forall(x: T) { p(x) = q(x) } {
        exists_unique_congr(p, q)
        exists_unique(q)
        choose_witness_unique_spec(p)
        p(choose_witness(p))
        q(choose_witness(p))
        choose_witness_unique_eq(q, choose_witness(p))
        choose_witness(q) = choose_witness(p)
        choose_witness(p) = choose_witness(q)
    }
}

/// The default-valued chosen witness satisfies the predicate when a witness exists.
theorem choose_or_default_spec[T](p: T -> Bool, default: T) {
    exists(x: T) {
        p(x)
    } implies p(choose_or_default(p, default))
}

/// The default-valued chosen witness is the default when no witness exists.
theorem choose_or_default_eq_default[T](p: T -> Bool, default: T) {
    not exists(x: T) {
        p(x)
    } implies choose_or_default(p, default) = default
}

/// The predicate that an element lies in a list and satisfies a condition.
define list_witness_predicate[T](items: List[T], p: T -> Bool, x: T) -> Bool {
    items.contains(x) and p(x)
}

/// A list witness belongs to the list.
theorem list_witness_contains[T](items: List[T], p: T -> Bool, x: T) {
    list_witness_predicate(items, p, x) implies items.contains(x)
}

/// A list witness satisfies the predicate.
theorem list_witness_property[T](items: List[T], p: T -> Bool, x: T) {
    list_witness_predicate(items, p, x) implies p(x)
}

/// List membership together with the predicate gives a list witness.
theorem list_witness_intro[T](items: List[T], p: T -> Bool, x: T) {
    items.contains(x) and p(x) implies list_witness_predicate(items, p, x)
}

/// A concrete list witness gives existence of a list witness.
theorem list_witness_exists_of_witness[T](items: List[T], p: T -> Bool, x: T) {
    list_witness_predicate(items, p, x) implies exists(y: T) {
        list_witness_predicate(items, p, y)
    }
} by {
    if list_witness_predicate(items, p, x) {
        exists_intro(list_witness_predicate(items, p), x)
    }
}

/// A chosen element of a list satisfying a condition, with a specified default otherwise.
let choose_from_list_or_default[T](items: List[T], p: T -> Bool, default: T) -> result: T satisfy {
    (exists(x: T) {
        list_witness_predicate(items, p, x)
    } implies list_witness_predicate(items, p, result)) and
    (not exists(x: T) {
        list_witness_predicate(items, p, x)
    } implies result = default)
} by {
    if exists(x: T) { list_witness_predicate(items, p, x) } {
        let x: T satisfy {
            list_witness_predicate(items, p, x)
        }
    } else {
        let x: T = default
    }
}

/// The chosen list element satisfies the list witness predicate when such an element exists.
theorem choose_from_list_or_default_spec[T](items: List[T], p: T -> Bool, default: T) {
    exists(x: T) {
        list_witness_predicate(items, p, x)
    } implies list_witness_predicate(items, p, choose_from_list_or_default(items, p, default))
}

/// The chosen list element belongs to the list when such an element exists.
theorem choose_from_list_or_default_contains[T](items: List[T], p: T -> Bool, default: T) {
    exists(x: T) {
        list_witness_predicate(items, p, x)
    } implies items.contains(choose_from_list_or_default(items, p, default))
} by {
    if exists(x: T) { list_witness_predicate(items, p, x) } {
        choose_from_list_or_default_spec(items, p, default)
        list_witness_predicate(items, p, choose_from_list_or_default(items, p, default))
        list_witness_contains(items, p, choose_from_list_or_default(items, p, default))
    }
}

/// The chosen list element satisfies the condition when such an element exists.
theorem choose_from_list_or_default_property[T](items: List[T], p: T -> Bool, default: T) {
    exists(x: T) {
        list_witness_predicate(items, p, x)
    } implies p(choose_from_list_or_default(items, p, default))
} by {
    if exists(x: T) { list_witness_predicate(items, p, x) } {
        choose_from_list_or_default_spec(items, p, default)
        list_witness_predicate(items, p, choose_from_list_or_default(items, p, default))
        list_witness_property(items, p, choose_from_list_or_default(items, p, default))
    }
}

/// The default is chosen when no list element satisfies the condition.
theorem choose_from_list_or_default_eq_default[T](items: List[T], p: T -> Bool, default: T) {
    not exists(x: T) {
        list_witness_predicate(items, p, x)
    } implies choose_from_list_or_default(items, p, default) = default
}

/// A unique list witness determines the chosen element.
theorem choose_from_list_or_default_unique_eq[T](items: List[T], p: T -> Bool, default: T, x: T) {
    exists_unique(list_witness_predicate(items, p)) and list_witness_predicate(items, p, x)
    implies choose_from_list_or_default(items, p, default) = x
} by {
    if exists_unique(list_witness_predicate(items, p)) and list_witness_predicate(items, p, x) {
        exists_unique_exists(list_witness_predicate(items, p))
        choose_from_list_or_default_spec(items, p, default)
        list_witness_predicate(items, p, choose_from_list_or_default(items, p, default))
        exists_unique_eq(list_witness_predicate(items, p),
            choose_from_list_or_default(items, p, default), x)
    }
}

/// Changing the default does not change the chosen list element when a unique list witness exists.
theorem choose_from_list_or_default_unique_default_eq[T](items: List[T], p: T -> Bool, a: T, b: T) {
    exists_unique(list_witness_predicate(items, p)) implies
    choose_from_list_or_default(items, p, a) = choose_from_list_or_default(items, p, b)
} by {
    if exists_unique(list_witness_predicate(items, p)) {
        exists_unique_exists(list_witness_predicate(items, p))
        choose_from_list_or_default_spec(items, p, a)
        choose_from_list_or_default_spec(items, p, b)
        list_witness_predicate(items, p, choose_from_list_or_default(items, p, a))
        list_witness_predicate(items, p, choose_from_list_or_default(items, p, b))
        exists_unique_eq(list_witness_predicate(items, p),
            choose_from_list_or_default(items, p, a),
            choose_from_list_or_default(items, p, b))
    }
}

/// The predicate that an element lies in a set and satisfies a condition.
define set_witness_predicate[T](s: Set[T], p: T -> Bool, x: T) -> Bool {
    s.contains(x) and p(x)
}

/// A set witness belongs to the set.
theorem set_witness_contains[T](s: Set[T], p: T -> Bool, x: T) {
    set_witness_predicate(s, p, x) implies s.contains(x)
}

/// A set witness satisfies the predicate.
theorem set_witness_property[T](s: Set[T], p: T -> Bool, x: T) {
    set_witness_predicate(s, p, x) implies p(x)
}

/// Set membership together with the predicate gives a set witness.
theorem set_witness_intro[T](s: Set[T], p: T -> Bool, x: T) {
    s.contains(x) and p(x) implies set_witness_predicate(s, p, x)
}

/// A concrete set witness gives existence of a set witness.
theorem set_witness_exists_of_witness[T](s: Set[T], p: T -> Bool, x: T) {
    set_witness_predicate(s, p, x) implies exists(y: T) {
        set_witness_predicate(s, p, y)
    }
} by {
    if set_witness_predicate(s, p, x) {
        exists_intro(set_witness_predicate(s, p), x)
    }
}

/// A chosen element of a set satisfying a condition, with a specified default otherwise.
let choose_from_set_or_default[T](s: Set[T], p: T -> Bool, default: T) -> result: T satisfy {
    (exists(x: T) {
        set_witness_predicate(s, p, x)
    } implies set_witness_predicate(s, p, result)) and
    (not exists(x: T) {
        set_witness_predicate(s, p, x)
    } implies result = default)
} by {
    if exists(x: T) { set_witness_predicate(s, p, x) } {
        let x: T satisfy {
            set_witness_predicate(s, p, x)
        }
    } else {
        let x: T = default
    }
}

/// The chosen set element satisfies the set witness predicate when such an element exists.
theorem choose_from_set_or_default_spec[T](s: Set[T], p: T -> Bool, default: T) {
    exists(x: T) {
        set_witness_predicate(s, p, x)
    } implies set_witness_predicate(s, p, choose_from_set_or_default(s, p, default))
}

/// The chosen set element belongs to the set when such an element exists.
theorem choose_from_set_or_default_contains[T](s: Set[T], p: T -> Bool, default: T) {
    exists(x: T) {
        set_witness_predicate(s, p, x)
    } implies s.contains(choose_from_set_or_default(s, p, default))
} by {
    if exists(x: T) { set_witness_predicate(s, p, x) } {
        choose_from_set_or_default_spec(s, p, default)
        set_witness_predicate(s, p, choose_from_set_or_default(s, p, default))
        set_witness_contains(s, p, choose_from_set_or_default(s, p, default))
    }
}

/// The chosen set element satisfies the condition when such an element exists.
theorem choose_from_set_or_default_property[T](s: Set[T], p: T -> Bool, default: T) {
    exists(x: T) {
        set_witness_predicate(s, p, x)
    } implies p(choose_from_set_or_default(s, p, default))
} by {
    if exists(x: T) { set_witness_predicate(s, p, x) } {
        choose_from_set_or_default_spec(s, p, default)
        set_witness_predicate(s, p, choose_from_set_or_default(s, p, default))
        set_witness_property(s, p, choose_from_set_or_default(s, p, default))
    }
}

/// The default is chosen when no set element satisfies the condition.
theorem choose_from_set_or_default_eq_default[T](s: Set[T], p: T -> Bool, default: T) {
    not exists(x: T) {
        set_witness_predicate(s, p, x)
    } implies choose_from_set_or_default(s, p, default) = default
}

/// A unique set witness determines the chosen element.
theorem choose_from_set_or_default_unique_eq[T](s: Set[T], p: T -> Bool, default: T, x: T) {
    exists_unique(set_witness_predicate(s, p)) and set_witness_predicate(s, p, x)
    implies choose_from_set_or_default(s, p, default) = x
} by {
    if exists_unique(set_witness_predicate(s, p)) and set_witness_predicate(s, p, x) {
        exists_unique_exists(set_witness_predicate(s, p))
        choose_from_set_or_default_spec(s, p, default)
        set_witness_predicate(s, p, choose_from_set_or_default(s, p, default))
        exists_unique_eq(set_witness_predicate(s, p),
            choose_from_set_or_default(s, p, default), x)
    }
}

/// Changing the default does not change the chosen set element when a unique set witness exists.
theorem choose_from_set_or_default_unique_default_eq[T](s: Set[T], p: T -> Bool, a: T, b: T) {
    exists_unique(set_witness_predicate(s, p)) implies
    choose_from_set_or_default(s, p, a) = choose_from_set_or_default(s, p, b)
} by {
    if exists_unique(set_witness_predicate(s, p)) {
        exists_unique_exists(set_witness_predicate(s, p))
        choose_from_set_or_default_spec(s, p, a)
        choose_from_set_or_default_spec(s, p, b)
        set_witness_predicate(s, p, choose_from_set_or_default(s, p, a))
        set_witness_predicate(s, p, choose_from_set_or_default(s, p, b))
        exists_unique_eq(set_witness_predicate(s, p),
            choose_from_set_or_default(s, p, a),
            choose_from_set_or_default(s, p, b))
    }
}

/// The default-valued chosen witness is equal to any unique witness.
theorem choose_or_default_unique_eq[T](p: T -> Bool, default: T, x: T) {
    exists_unique(p) and p(x) implies choose_or_default(p, default) = x
} by {
    if exists_unique(p) and p(x) {
        exists_unique_exists(p)
        choose_or_default_spec(p, default)
        p(choose_or_default(p, default))
        exists_unique_eq(p, choose_or_default(p, default), x)
    }
}

/// Changing the default does not change the chosen value when a unique witness exists.
theorem choose_or_default_unique_default_eq[T](p: T -> Bool, a: T, b: T) {
    exists_unique(p) implies choose_or_default(p, a) = choose_or_default(p, b)
} by {
    if exists_unique(p) {
        exists_unique_exists(p)
        choose_or_default_spec(p, a)
        choose_or_default_spec(p, b)
        p(choose_or_default(p, a))
        p(choose_or_default(p, b))
        exists_unique_eq(p, choose_or_default(p, a), choose_or_default(p, b))
    }
}

/// Equal predicates have the same default-valued chosen witness.
theorem choose_or_default_eq_of_predicate_eq[T](p: T -> Bool, q: T -> Bool, default: T) {
    p = q implies choose_or_default(p, default) = choose_or_default(q, default)
}

/// A default-valued chosen witness transports along equality of predicates when a witness exists.
theorem choose_or_default_spec_of_predicate_eq[T](p: T -> Bool, q: T -> Bool, default: T) {
    p = q and exists(x: T) {
        p(x)
    } implies q(choose_or_default(p, default))
} by {
    if p = q and exists(x: T) { p(x) } {
        choose_or_default_spec(p, default)
    }
}

/// A default-valued chosen witness for an equal uniquely satisfiable predicate is the given witness.
theorem choose_or_default_unique_eq_of_predicate_eq[T](p: T -> Bool, q: T -> Bool, default: T, x: T) {
    p = q and exists_unique(p) and q(x) implies choose_or_default(p, default) = x
} by {
    if p = q and exists_unique(p) and q(x) {
        p(x)
        choose_or_default_unique_eq(p, default, x)
    }
}

/// Unique existence transports along equality of predicates.
theorem exists_unique_of_predicate_eq[T](p: T -> Bool, q: T -> Bool) {
    p = q and exists_unique(p) implies exists_unique(q)
}

/// A predicate on pairs induced by a binary predicate.
define pair_witness_predicate[T, U](p: (T, U) -> Bool, pair: Pair[T, U]) -> Bool {
    p(pair.first, pair.second)
}

/// A pair witness gives the original binary predicate on its components.
theorem pair_witness_property[T, U](p: (T, U) -> Bool, pair: Pair[T, U]) {
    pair_witness_predicate(p, pair) implies p(pair.first, pair.second)
}

/// A binary predicate on a pair of elements gives a pair witness.
theorem pair_witness_intro[T, U](p: (T, U) -> Bool, x: T, y: U) {
    p(x, y) implies pair_witness_predicate(p, Pair.new(x, y))
} by {
    if p(x, y) {
        let pair = Pair.new(x, y)
        pair.first = x
        pair.second = y
        p(pair.first, pair.second)
        pair_witness_predicate(p, pair)
    }
}

/// A concrete pair witness gives existence of a pair witness.
theorem pair_witness_exists_of_witness[T, U](p: (T, U) -> Bool, pair: Pair[T, U]) {
    pair_witness_predicate(p, pair) implies exists(q: Pair[T, U]) {
        pair_witness_predicate(p, q)
    }
} by {
    if pair_witness_predicate(p, pair) {
        exists_intro(pair_witness_predicate(p), pair)
    }
}

/// A nested witness gives a pair witness.
theorem exists_pair_of_exists_nested[T, U](p: (T, U) -> Bool) {
    exists(x: T) {
        exists(y: U) {
            p(x, y)
        }
    } implies exists(pair: Pair[T, U]) {
        pair_witness_predicate(p, pair)
    }
} by {
    if exists(x: T) { exists(y: U) { p(x, y) } } {
        let x: T satisfy {
            exists(y: U) {
                p(x, y)
            }
        }
        let y: U satisfy {
            p(x, y)
        }
        pair_witness_intro(p, x, y)
        exists(pair: Pair[T, U]) {
            pair = Pair.new(x, y) and pair_witness_predicate(p, pair)
        }
    }
}

/// A pair witness gives nested witnesses.
theorem exists_nested_of_exists_pair[T, U](p: (T, U) -> Bool) {
    exists(pair: Pair[T, U]) {
        pair_witness_predicate(p, pair)
    } implies exists(x: T) {
        exists(y: U) {
            p(x, y)
        }
    }
} by {
    if exists(pair: Pair[T, U]) { pair_witness_predicate(p, pair) } {
        let pair: Pair[T, U] satisfy {
            pair_witness_predicate(p, pair)
        }
        pair_witness_property(p, pair)
        exists(x: T) {
            x = pair.first and exists(y: U) {
                y = pair.second and p(x, y)
            }
        }
    }
}

/// Nested existence is equivalent to existence of a pair witness.
theorem exists_nested_iff_exists_pair[T, U](p: (T, U) -> Bool) {
    (exists(x: T) {
        exists(y: U) {
            p(x, y)
        }
    }) = (exists(pair: Pair[T, U]) {
        pair_witness_predicate(p, pair)
    })
} by {
    exists_pair_of_exists_nested(p)
    exists_nested_of_exists_pair(p)
}

/// Pair-witness unique existence gives uniqueness of first nested witnesses.
theorem exists_pair_unique_nested_first_eq[T, U](p: (T, U) -> Bool, x1: T, y1: U, x2: T, y2: U) {
    exists_unique(pair_witness_predicate(p)) and p(x1, y1) and p(x2, y2)
    implies x1 = x2
} by {
    if exists_unique(pair_witness_predicate(p)) and p(x1, y1) and p(x2, y2) {
        let pair1 = Pair.new(x1, y1)
        let pair2 = Pair.new(x2, y2)
        pair_witness_intro(p, x1, y1)
        pair_witness_intro(p, x2, y2)
        exists_unique_eq(pair_witness_predicate(p), pair1, pair2)
        pair1 = pair2
        pair1.first = pair2.first
        x1 = x2
    }
}

/// Pair-witness unique existence gives uniqueness of second nested witnesses.
theorem exists_pair_unique_nested_second_eq[T, U](p: (T, U) -> Bool, x1: T, y1: U, x2: T, y2: U) {
    exists_unique(pair_witness_predicate(p)) and p(x1, y1) and p(x2, y2)
    implies y1 = y2
} by {
    if exists_unique(pair_witness_predicate(p)) and p(x1, y1) and p(x2, y2) {
        let pair1 = Pair.new(x1, y1)
        let pair2 = Pair.new(x2, y2)
        pair_witness_intro(p, x1, y1)
        pair_witness_intro(p, x2, y2)
        exists_unique_eq(pair_witness_predicate(p), pair1, pair2)
        pair1 = pair2
        pair1.second = pair2.second
        y1 = y2
    }
}

/// The chosen witness satisfies a uniquely satisfiable predicate.
theorem choose_unique_spec[T: Inhabited](p: T -> Bool) {
    exists_unique(p) implies p(choose_of_exists(p))
} by {
    if exists_unique(p) {
        exists_unique_exists(p)
        exists(x: T) {
            p(x)
        }
        choose_of_exists_spec(p)
    }
}

/// The chosen witness is equal to any witness of a uniquely satisfiable predicate.
theorem choose_unique_eq[T: Inhabited](p: T -> Bool, x: T) {
    exists_unique(p) and p(x) implies choose_of_exists(p) = x
} by {
    if exists_unique(p) and p(x) {
        choose_unique_spec(p)
        p(choose_of_exists(p))
        exists_unique_eq(p, choose_of_exists(p), x)
    }
}

/// Equivalent uniquely satisfiable predicates have the same chosen witness.
theorem choose_unique_congr[T: Inhabited](p: T -> Bool, q: T -> Bool) {
    exists_unique(p) and forall(x: T) {
        p(x) = q(x)
    } implies choose_of_exists(p) = choose_of_exists(q)
} by {
    if exists_unique(p) and forall(x: T) { p(x) = q(x) } {
        exists_unique_congr(p, q)
        exists_unique(q)
        choose_unique_spec(p)
        p(choose_of_exists(p))
        q(choose_of_exists(p))
        choose_unique_eq(q, choose_of_exists(p))
        choose_of_exists(q) = choose_of_exists(p)
    }
}

/// Equal predicates have the same chosen witness.
theorem choose_of_exists_eq_of_predicate_eq[T: Inhabited](p: T -> Bool, q: T -> Bool) {
    p = q implies choose_of_exists(p) = choose_of_exists(q)
}

/// A chosen witness transports along equality of predicates when the target has a witness.
theorem choose_of_exists_spec_of_predicate_eq[T: Inhabited](p: T -> Bool, q: T -> Bool) {
    p = q and exists(x: T) {
        p(x)
    } implies q(choose_of_exists(p))
} by {
    if p = q and exists(x: T) { p(x) } {
        choose_of_exists_spec(p)
    }
}

/// A chosen witness for the target predicate satisfies the source predicate under equality.
theorem choose_of_exists_spec_of_eq_target[T: Inhabited](p: T -> Bool, q: T -> Bool) {
    p = q and exists(x: T) {
        q(x)
    } implies p(choose_of_exists(q))
} by {
    if p = q and exists(x: T) { q(x) } {
        choose_of_exists_spec(q)
    }
}

/// Equal predicates have equivalent unique-existence properties.
theorem exists_unique_eq_of_predicate_eq[T](p: T -> Bool, q: T -> Bool) {
    p = q implies exists_unique(p) = exists_unique(q)
}

/// A chosen witness for an equal uniquely satisfiable predicate is the given witness.
theorem choose_unique_eq_of_predicate_eq[T: Inhabited](p: T -> Bool, q: T -> Bool, x: T) {
    p = q and exists_unique(p) and q(x) implies choose_of_exists(p) = x
} by {
    if p = q and exists_unique(p) and q(x) {
        p(x)
        choose_unique_eq(p, x)
    }
}
