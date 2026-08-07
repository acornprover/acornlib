/// Reusable rewrite lemmas for `and`, `or`, `not`, and implication.

/// Conjunction gives its left conjunct.
theorem and_left(a: Bool, b: Bool) {
    a and b implies a
}

/// Conjunction gives its right conjunct.
theorem and_right(a: Bool, b: Bool) {
    a and b implies b
}

/// Two propositions give their conjunction.
theorem and_intro(a: Bool, b: Bool) {
    a and b implies a and b
}

/// Conjunction is commutative.
theorem and_comm(a: Bool, b: Bool) {
    (a and b) = (b and a)
}

/// Conjunction is associative.
theorem and_assoc(a: Bool, b: Bool, c: Bool) {
    ((a and b) and c) = (a and (b and c))
}

/// Conjunction is idempotent.
theorem and_self(a: Bool) {
    (a and a) = a
}

/// `true` is the identity for conjunction.
theorem and_true(a: Bool) {
    (a and true) = a
}

/// Conjunction with `true` on the left is identity.
theorem true_and(a: Bool) {
    (true and a) = a
}

/// `false` is absorbing for conjunction.
theorem and_false(a: Bool) {
    (a and false) = false
}

/// `false` on the left is absorbing for conjunction.
theorem false_and(a: Bool) {
    (false and a) = false
}

/// Disjunction is commutative.
theorem or_comm(a: Bool, b: Bool) {
    (a or b) = (b or a)
}

/// Disjunction is associative.
theorem or_assoc(a: Bool, b: Bool, c: Bool) {
    ((a or b) or c) = (a or (b or c))
}

/// The left disjunct gives the disjunction.
theorem or_intro_left(a: Bool, b: Bool) {
    a implies a or b
}

/// The right disjunct gives the disjunction.
theorem or_intro_right(a: Bool, b: Bool) {
    b implies a or b
}

/// Disjunction is idempotent.
theorem or_self(a: Bool) {
    (a or a) = a
}

/// `true` is absorbing for disjunction.
theorem or_true(a: Bool) {
    (a or true) = true
}

/// `true` on the left is absorbing for disjunction.
theorem true_or(a: Bool) {
    (true or a) = true
}

/// `false` is the identity for disjunction.
theorem or_false(a: Bool) {
    (a or false) = a
}

/// `false` on the left is identity for disjunction.
theorem false_or(a: Bool) {
    (false or a) = a
}

/// Negating `true` gives `false`.
theorem not_true {
    (not true) = false
}

/// Negating `false` gives `true`.
theorem not_false {
    (not false) = true
}

/// Double negation eliminates.
theorem not_not(a: Bool) {
    (not not a) = a
}

/// Double negation eliminates classically.
theorem not_not_elim(a: Bool) {
    not not a implies a
}

/// A proposition implies its double negation.
theorem not_not_intro(a: Bool) {
    a implies not not a
}

/// Conjunction with one's own negation is `false`.
theorem and_not_self(a: Bool) {
    (a and not a) = false
}

/// Disjunction with one's own negation is `true`.
theorem or_not_self(a: Bool) {
    (a or not a) = true
}

/// De Morgan: negation of a conjunction.
theorem not_and(a: Bool, b: Bool) {
    (not (a and b)) = (not a or not b)
}

/// De Morgan: negation of a disjunction.
theorem not_or(a: Bool, b: Bool) {
    (not (a or b)) = (not a and not b)
}

/// `true` antecedent: `(true implies p) = p`.
theorem true_implies(a: Bool) {
    (true implies a) = a
}

/// `false` antecedent: `(false implies p) = true`.
theorem false_implies(a: Bool) {
    (false implies a) = true
}

/// `true` consequent: `(p implies true) = true`.
theorem implies_true(a: Bool) {
    (a implies true) = true
}

/// `false` consequent: `(p implies false) = not p`.
theorem implies_false(a: Bool) {
    (a implies false) = not a
}

/// Self-implication is `true`.
theorem implies_self(a: Bool) {
    (a implies a) = true
}

/// Implication is disjunction with the antecedent negated.
theorem implies_eq_not_or(a: Bool, b: Bool) {
    (a implies b) = (not a or b)
}

/// Negating an implication.
theorem not_implies(a: Bool, b: Bool) {
    (not (a implies b)) = (a and not b)
}

/// A contradiction proves any proposition.
theorem false_elim(a: Bool) {
    false implies a
} by {
    if false {
        a
    }
}

/// A proposition that implies contradiction is false.
theorem not_intro(a: Bool) {
    (a implies false) implies not a
} by {
    if a implies false {
        if a {
            false
        }
    }
}

/// A proposition and its negation cannot both hold.
theorem not_and_self(a: Bool) {
    not (a and not a)
} by {
    if a and not a {
        if not a {
            false
        }
    }
}

/// A negation and its proposition cannot both hold.
theorem not_self_and(a: Bool) {
    not (not a and a)
} by {
    if not a and a {
        if not a {
            false
        }
    }
}

/// Proof by contradiction is valid classically.
theorem by_contradiction(a: Bool) {
    (not a implies false) implies a
} by {
    if not a implies false {
        if not a {
            false
        }
        a
    }
}

/// Every proposition satisfies excluded middle.
theorem excluded_middle(a: Bool) {
    a or not a
} by {
    if a {
        a or not a
    } else {
        a or not a
    }
}

/// Contraposition.
theorem contrapose(a: Bool, b: Bool) {
    (a implies b) and not b implies not a
}

/// Contrapositive implication is equivalent to the original implication.
theorem contrapose_iff(a: Bool, b: Bool) {
    (a implies b) = (not b implies not a)
}

/// Contrapositive form of implication.
theorem implies_contrapositive(a: Bool, b: Bool) {
    (a implies b) = (not b implies not a)
}

/// A proposition equal to true holds.
theorem eq_true_elim(a: Bool) {
    a = true implies a
}

/// A true proposition is equal to true.
theorem eq_true_intro(a: Bool) {
    a implies a = true
}

/// A proposition equal to false does not hold.
theorem eq_false_elim(a: Bool) {
    a = false implies not a
}

/// A false proposition is equal to false.
theorem eq_false_intro(a: Bool) {
    not a implies a = false
}

/// True if two propositions imply each other.
define iff_bool(a: Bool, b: Bool) -> Bool {
    (a implies b) and (b implies a)
}

/// Logical equivalence is reflexive.
theorem iff_bool_refl(a: Bool) {
    iff_bool(a, a)
}

/// Logical equivalence is symmetric.
theorem iff_bool_symm(a: Bool, b: Bool) {
    iff_bool(a, b) implies iff_bool(b, a)
}

/// Logical equivalence is transitive.
theorem iff_bool_trans(a: Bool, b: Bool, c: Bool) {
    iff_bool(a, b) and iff_bool(b, c) implies iff_bool(a, c)
} by {
    if iff_bool(a, b) and iff_bool(b, c) {
        if a {
            b
            c
        }
        if c {
            b
            a
        }
        iff_bool(a, c)
    }
}

/// Logical equivalence gives the forward implication.
theorem iff_bool_forward(a: Bool, b: Bool) {
    iff_bool(a, b) implies (a implies b)
}

/// Logical equivalence gives the backward implication.
theorem iff_bool_backward(a: Bool, b: Bool) {
    iff_bool(a, b) implies (b implies a)
}

/// Equal propositions are logically equivalent.
theorem eq_imp_iff_bool(a: Bool, b: Bool) {
    a = b implies iff_bool(a, b)
} by {
    if a = b {
        if a {
            b
        }
        if b {
            a
        }
        iff_bool(a, b)
    }
}

/// Logically equivalent propositions are equal.
theorem iff_bool_imp_eq(a: Bool, b: Bool) {
    iff_bool(a, b) implies a = b
} by {
    if iff_bool(a, b) {
        if a {
            b
        }
        if b {
            a
        }
    }
}

/// Logical equivalence is the same as equality of propositions.
theorem iff_bool_eq_eq(a: Bool, b: Bool) {
    iff_bool(a, b) = (a = b)
} by {
    iff_bool_imp_eq(a, b)
    eq_imp_iff_bool(a, b)
}

/// Equality of propositions is the same as logical equivalence.
theorem eq_eq_iff_bool(a: Bool, b: Bool) {
    (a = b) = iff_bool(a, b)
} by {
    iff_bool_eq_eq(a, b)
}

/// Logically equivalent propositions have equivalent negations.
theorem iff_bool_not(a: Bool, b: Bool) {
    iff_bool(a, b) implies iff_bool(not a, not b)
} by {
    if iff_bool(a, b) {
        if not a {
            if b {
                false
            }
            not b
        }
        if not b {
            if a {
                false
            }
            not a
        }
        iff_bool(not a, not b)
    }
}

/// Logical equivalence may rewrite the left side of a conjunction.
theorem iff_bool_and_left(a: Bool, b: Bool, c: Bool) {
    iff_bool(a, b) implies iff_bool(a and c, b and c)
} by {
    if iff_bool(a, b) {
        if a and c {
            b and c
        }
        if b and c {
            a and c
        }
        iff_bool(a and c, b and c)
    }
}

/// Logical equivalence may rewrite the right side of a conjunction.
theorem iff_bool_and_right(a: Bool, b: Bool, c: Bool) {
    iff_bool(a, b) implies iff_bool(c and a, c and b)
} by {
    if iff_bool(a, b) {
        if c and a {
            c and b
        }
        if c and b {
            c and a
        }
        iff_bool(c and a, c and b)
    }
}

/// Logical equivalence may rewrite both sides of a conjunction.
theorem iff_bool_and(a: Bool, b: Bool, c: Bool, d: Bool) {
    iff_bool(a, b) and iff_bool(c, d) implies iff_bool(a and c, b and d)
} by {
    if iff_bool(a, b) and iff_bool(c, d) {
        if a and c {
            b and d
        }
        if b and d {
            a and c
        }
        (a and c implies b and d) and (b and d implies a and c)
        iff_bool(a and c, b and d)
    }
}

/// Logical equivalence may rewrite the left side of a disjunction.
theorem iff_bool_or_left(a: Bool, b: Bool, c: Bool) {
    iff_bool(a, b) implies iff_bool(a or c, b or c)
} by {
    if iff_bool(a, b) {
        if a or c {
            if a {
                b or c
            } else {
                b or c
            }
        }
        if b or c {
            if b {
                a or c
            } else {
                a or c
            }
        }
        iff_bool(a or c, b or c)
    }
}

/// Logical equivalence may rewrite the right side of a disjunction.
theorem iff_bool_or_right(a: Bool, b: Bool, c: Bool) {
    iff_bool(a, b) implies iff_bool(c or a, c or b)
} by {
    if iff_bool(a, b) {
        if c or a {
            if c {
                c or b
            } else {
                c or b
            }
        }
        if c or b {
            if c {
                c or a
            } else {
                c or a
            }
        }
        iff_bool(c or a, c or b)
    }
}

/// Logical equivalence may rewrite both sides of a disjunction.
theorem iff_bool_or(a: Bool, b: Bool, c: Bool, d: Bool) {
    iff_bool(a, b) and iff_bool(c, d) implies iff_bool(a or c, b or d)
} by {
    if iff_bool(a, b) and iff_bool(c, d) {
        if a or c {
            if a {
                b or d
            } else {
                b or d
            }
        }
        if b or d {
            if b {
                a or c
            } else {
                a or c
            }
        }
        (a or c implies b or d) and (b or d implies a or c)
        iff_bool(a or c, b or d)
    }
}

/// Logical equivalence may rewrite the antecedent of an implication.
theorem iff_bool_implies_left(a: Bool, b: Bool, c: Bool) {
    iff_bool(a, b) implies iff_bool(a implies c, b implies c)
} by {
    if iff_bool(a, b) {
        if a implies c {
            if b {
                c
            }
        }
        if b implies c {
            if a {
                c
            }
        }
        iff_bool(a implies c, b implies c)
    }
}

/// Logical equivalence may rewrite the consequent of an implication.
theorem iff_bool_implies_right(a: Bool, b: Bool, c: Bool) {
    iff_bool(a, b) implies iff_bool(c implies a, c implies b)
} by {
    if iff_bool(a, b) {
        if c implies a {
            if c {
                b
            }
        }
        if c implies b {
            if c {
                a
            }
        }
        iff_bool(c implies a, c implies b)
    }
}

/// Logical equivalence may rewrite both sides of an implication.
theorem iff_bool_implies(a: Bool, b: Bool, c: Bool, d: Bool) {
    iff_bool(a, b) and iff_bool(c, d) implies iff_bool(a implies c, b implies d)
} by {
    if iff_bool(a, b) and iff_bool(c, d) {
        if a implies c {
            if b {
                d
            }
        }
        if b implies d {
            if a {
                c
            }
        }
        iff_bool(a implies c, b implies d)
    }
}

/// Pointwise logical equivalence rewrites universal quantification.
theorem iff_bool_forall[T](p: T -> Bool, q: T -> Bool) {
    (forall(x: T) { iff_bool(p(x), q(x)) }) implies
    iff_bool(forall(x: T) { p(x) }, forall(x: T) { q(x) })
} by {
    if forall(x: T) { iff_bool(p(x), q(x)) } {
        if forall(x: T) { p(x) } {
            forall(x: T) {
                q(x)
            }
        }
        if forall(x: T) { q(x) } {
            forall(x: T) {
                p(x)
            }
        }
        iff_bool(forall(x: T) { p(x) }, forall(x: T) { q(x) })
    }
}

/// Pointwise logical equivalence gives the forward existential implication.
theorem iff_bool_exists_forward[T](p: T -> Bool, q: T -> Bool) {
    (forall(x: T) { iff_bool(p(x), q(x)) }) implies
    (exists(x: T) { p(x) } implies exists(x: T) { q(x) })
} by {
    if forall(x: T) { iff_bool(p(x), q(x)) } {
        if exists(x: T) { p(x) } {
            let x: T satisfy {
                p(x)
            }
            exists(witness: T) {
                witness = x and q(witness)
            }
        }
    }
}

/// Pointwise logical equivalence gives the backward existential implication.
theorem iff_bool_exists_backward[T](p: T -> Bool, q: T -> Bool) {
    (forall(x: T) { iff_bool(p(x), q(x)) }) implies
    (exists(x: T) { q(x) } implies exists(x: T) { p(x) })
} by {
    if forall(x: T) { iff_bool(p(x), q(x)) } {
        if exists(x: T) { q(x) } {
            let x: T satisfy {
                q(x)
            }
            exists(witness: T) {
                witness = x and p(witness)
            }
        }
    }
}

/// Pointwise logical equivalence rewrites existential quantification.
theorem iff_bool_exists[T](p: T -> Bool, q: T -> Bool) {
    (forall(x: T) { iff_bool(p(x), q(x)) }) implies
    iff_bool(exists(x: T) { p(x) }, exists(x: T) { q(x) })
} by {
    if forall(x: T) { iff_bool(p(x), q(x)) } {
        iff_bool_exists_forward(p, q)
        iff_bool_exists_backward(p, q)
        exists(x: T) { q(x) } implies exists(x: T) { p(x) }
        iff_bool(exists(x: T) { p(x) }, exists(x: T) { q(x) })
    }
}

/// Conjunction distributes over disjunction on the left.
theorem and_or_distrib_left(a: Bool, b: Bool, c: Bool) {
    (a and (b or c)) = ((a and b) or (a and c))
}

/// Disjunction distributes over conjunction on the left.
theorem or_and_distrib_left(a: Bool, b: Bool, c: Bool) {
    (a or (b and c)) = ((a or b) and (a or c))
}

/// Conjunction distributes over disjunction on the right.
theorem and_or_distrib_right(a: Bool, b: Bool, c: Bool) {
    ((a or b) and c) = ((a and c) or (b and c))
} by {
    if (a or b) and c {
        if a {
            (a and c) or (b and c)
        } else {
            (a and c) or (b and c)
        }
    }
    if (a and c) or (b and c) {
        (a or b) and c
    }
}

/// Disjunction distributes over conjunction on the right.
theorem or_and_distrib_right(a: Bool, b: Bool, c: Bool) {
    ((a and b) or c) = ((a or c) and (b or c))
} by {
    if (a and b) or c {
        (a or c) and (b or c)
    }
    if (a or c) and (b or c) {
        if c {
            (a and b) or c
        } else {
            (a and b) or c
        }
    }
}

/// Implication into a conjunction is conjunction of implications.
theorem implies_and(a: Bool, b: Bool, c: Bool) {
    (a implies b and c) = ((a implies b) and (a implies c))
} by {
    if a implies b and c {
        (a implies b) and (a implies c)
    }
    if (a implies b) and (a implies c) {
        if a {
            b and c
        }
    }
}

/// Implication out of a disjunction is conjunction of implications.
theorem or_implies(a: Bool, b: Bool, c: Bool) {
    ((a or b) implies c) = ((a implies c) and (b implies c))
} by {
    if (a or b) implies c {
        (a implies c) and (b implies c)
    }
    if (a implies c) and (b implies c) {
        if a or b {
            c
        }
    }
}

/// An implication is a disjunction with the negated antecedent.
theorem implies_eq_or_not(a: Bool, b: Bool) {
    (a implies b) = (not a or b)
} by {
    if a implies b {
        if a {
            not a or b
        } else {
            not a or b
        }
    }
    if not a or b {
        if a {
            b
        }
    }
}


/// Negation reverses equality of propositions.
theorem not_eq_not(a: Bool, b: Bool) {
    a = b implies not a = not b
} by {
    if a = b {
        if not a {
            if b {
                false
            }
        }
        if not b {
            if a {
                false
            }
        }
    }
}

/// Equal propositions have the same implications out of them.
theorem eq_implies_left(a: Bool, b: Bool, c: Bool) {
    a = b implies (a implies c) = (b implies c)
} by {
    if a = b {
        if a implies c {
            if b {
                c
            }
        }
        if b implies c {
            if a {
                c
            }
        }
    }
}

/// Equal propositions have the same implications into them.
theorem eq_implies_right(a: Bool, b: Bool, c: Bool) {
    a = b implies (c implies a) = (c implies b)
} by {
    if a = b {
        if c implies a {
            if c {
                b
            }
        }
        if c implies b {
            if c {
                a
            }
        }
    }
}

/// Universal quantification distributes over conjunction.
theorem forall_and_distrib[T](p: T -> Bool, q: T -> Bool) {
    (forall(x: T) { p(x) and q(x) }) =
    ((forall(x: T) { p(x) }) and (forall(x: T) { q(x) }))
} by {
    if forall(x: T) { p(x) and q(x) } {
        forall(x: T) { p(x) }
        forall(x: T) { q(x) }
    }
    if (forall(x: T) { p(x) }) and (forall(x: T) { q(x) }) {
        forall(x: T) {
            p(x) and q(x)
        }
    }
}

/// A universal conjunction gives its left universal part.
theorem forall_and_left[T](p: T -> Bool, q: T -> Bool) {
    (forall(x: T) { p(x) and q(x) }) implies forall(x: T) { p(x) }
} by {
    if forall(x: T) { p(x) and q(x) } {
        forall(x: T) {
            p(x)
        }
    }
}

/// A universal conjunction gives its right universal part.
theorem forall_and_right[T](p: T -> Bool, q: T -> Bool) {
    (forall(x: T) { p(x) and q(x) }) implies forall(x: T) { q(x) }
} by {
    if forall(x: T) { p(x) and q(x) } {
        forall(x: T) {
            q(x)
        }
    }
}

/// Two universal facts give a universal conjunction.
theorem forall_and_intro[T](p: T -> Bool, q: T -> Bool) {
    (forall(x: T) { p(x) }) and (forall(x: T) { q(x) }) implies
    forall(x: T) { p(x) and q(x) }
} by {
    if (forall(x: T) { p(x) }) and (forall(x: T) { q(x) }) {
        forall(x: T) {
            p(x) and q(x)
        }
    }
}

/// An existential disjunction gives a disjunction of existentials.
theorem exists_or_forward[T](p: T -> Bool, q: T -> Bool) {
    exists(x: T) { p(x) or q(x) } implies
    (exists(x: T) { p(x) }) or (exists(x: T) { q(x) })
} by {
    if exists(x: T) { p(x) or q(x) } {
        let x: T satisfy {
            p(x) or q(x)
        }
        if p(x) {
            exists(witness: T) {
                witness = x and p(witness)
            }
        } else {
            exists(witness: T) {
                witness = x and q(witness)
            }
        }
    }
}

/// A left existential gives an existential disjunction.
theorem exists_or_backward_left[T](p: T -> Bool, q: T -> Bool) {
    exists(x: T) { p(x) } implies exists(x: T) { p(x) or q(x) }
} by {
    if exists(x: T) { p(x) } {
        let x: T satisfy {
            p(x)
        }
        exists(witness: T) {
            witness = x and (p(witness) or q(witness))
        }
    }
}

/// A right existential gives an existential disjunction.
theorem exists_or_backward_right[T](p: T -> Bool, q: T -> Bool) {
    exists(x: T) { q(x) } implies exists(x: T) { p(x) or q(x) }
} by {
    if exists(x: T) { q(x) } {
        let x: T satisfy {
            q(x)
        }
        exists(witness: T) {
            witness = x and (p(witness) or q(witness))
        }
    }
}

/// A disjunction of existentials gives an existential disjunction.
theorem exists_or_backward[T](p: T -> Bool, q: T -> Bool) {
    (exists(x: T) { p(x) }) or (exists(x: T) { q(x) }) implies
    exists(x: T) { p(x) or q(x) }
} by {
    if (exists(x: T) { p(x) }) or (exists(x: T) { q(x) }) {
        exists_or_backward_left(p, q)
        exists_or_backward_right(p, q)
    }
}

/// Existential quantification distributes over disjunction.
theorem exists_or_distrib[T](p: T -> Bool, q: T -> Bool) {
    (exists(x: T) { p(x) or q(x) }) =
    ((exists(x: T) { p(x) }) or (exists(x: T) { q(x) }))
} by {
    exists_or_forward(p, q)
    exists_or_backward(p, q)
}

/// A satisfying element gives existence.
theorem exists_intro[T](p: T -> Bool, x: T) {
    p(x) implies exists(y: T) {
        p(y)
    }
} by {
    if p(x) {
        exists(y: T) {
            y = x and p(y)
        }
    }
}

/// Nonexistence gives universal negation.
theorem not_exists_forward[T](p: T -> Bool) {
    not exists(x: T) { p(x) } implies forall(x: T) {
        not p(x)
    }
} by {
    if not exists(x: T) { p(x) } {
        forall(x: T) {
            if p(x) {
                false
            }
            not p(x)
        }
    }
}

/// Universal negation gives nonexistence.
theorem not_exists_backward[T](p: T -> Bool) {
    (forall(x: T) { not p(x) }) implies not exists(x: T) {
        p(x)
    }
} by {
    if forall(x: T) { not p(x) } {
        if exists(x: T) { p(x) } {
            let x: T satisfy {
                p(x)
            }
            false
        }
        not exists(x: T) {
            p(x)
        }
    }
}

/// Negating existence is universal negation.
theorem not_exists[T](p: T -> Bool) {
    (not exists(x: T) { p(x) }) = forall(x: T) {
        not p(x)
    }
} by {
    not_exists_forward(p)
    not_exists_backward(p)
}

/// A counterexample to a universal statement negates it.
theorem exists_not_imp_not_forall[T](p: T -> Bool) {
    exists(x: T) {
        not p(x)
    } implies not forall(x: T) {
        p(x)
    }
} by {
    if exists(x: T) { not p(x) } {
        let x: T satisfy {
            not p(x)
        }
        if forall(y: T) { p(y) } {
            false
        }
        not forall(y: T) {
            p(y)
        }
    }
}

/// Negating a universal statement gives a counterexample.
theorem not_forall_imp_exists_not[T](p: T -> Bool) {
    not forall(x: T) {
        p(x)
    } implies exists(x: T) {
        not p(x)
    }
} by {
    if not forall(x: T) { p(x) } {
        if not exists(x: T) { not p(x) } {
            not_exists_forward(function(x: T) { not p(x) })
            forall(x: T) {
                if not p(x) {
                    false
                }
                p(x)
            }
            false
        }
        exists(x: T) {
            not p(x)
        }
    }
}

/// Negating universal quantification is existential negation.
theorem not_forall[T](p: T -> Bool) {
    (not forall(x: T) { p(x) }) = exists(x: T) {
        not p(x)
    }
} by {
    exists_not_imp_not_forall(p)
    not_forall_imp_exists_not(p)
}

/// Universal truth is the same as no counterexample.
theorem forall_eq_not_exists_not[T](p: T -> Bool) {
    (forall(x: T) { p(x) }) = not exists(x: T) {
        not p(x)
    }
} by {
    if forall(x: T) { p(x) } {
        if exists(x: T) { not p(x) } {
            let x: T satisfy {
                not p(x)
            }
            false
        }
        not exists(x: T) {
            not p(x)
        }
    }
    if not exists(x: T) { not p(x) } {
        not_exists_forward(function(x: T) { not p(x) })
        forall(x: T) {
            if not p(x) {
                false
            }
            p(x)
        }
    }
}

/// Existence is the same as not all witnesses failing.
theorem exists_eq_not_forall_not[T](p: T -> Bool) {
    (exists(x: T) { p(x) }) = not forall(x: T) {
        not p(x)
    }
} by {
    if exists(x: T) { p(x) } {
        let x: T satisfy {
            p(x)
        }
        if forall(y: T) { not p(y) } {
            false
        }
        not forall(y: T) {
            not p(y)
        }
    }
    if not forall(x: T) { not p(x) } {
        not_forall_imp_exists_not(function(x: T) { not p(x) })
        let x: T satisfy {
            not not p(x)
        }
        if not p(x) {
            false
        }
        exists(witness: T) {
            witness = x and p(witness)
        }
    }
}

/// A fixed implication may be moved through universal quantification.
theorem forall_implies_const_left[T](a: Bool, p: T -> Bool) {
    (forall(x: T) { a implies p(x) }) = (a implies forall(x: T) { p(x) })
} by {
    if forall(x: T) { a implies p(x) } {
        if a {
            forall(x: T) {
                p(x)
            }
        }
    }
    if a implies forall(x: T) { p(x) } {
        forall(x: T) {
            a implies p(x)
        }
    }
}

/// Universal implication with a fixed antecedent gives implication into a universal.
theorem forall_implies_const_left_forward[T](a: Bool, p: T -> Bool) {
    (forall(x: T) { a implies p(x) }) implies (a implies forall(x: T) { p(x) })
} by {
    if forall(x: T) { a implies p(x) } {
        if a {
            forall(x: T) {
                p(x)
            }
        }
    }
}

/// Implication into a universal gives universal implication with a fixed antecedent.
theorem forall_implies_const_left_backward[T](a: Bool, p: T -> Bool) {
    (a implies forall(x: T) { p(x) }) implies forall(x: T) { a implies p(x) }
} by {
    if a implies forall(x: T) { p(x) } {
        forall(x: T) {
            a implies p(x)
        }
    }
}

/// A fixed consequent may be moved out of universal quantification as an existential antecedent.
theorem forall_implies_const_right[T](p: T -> Bool, a: Bool) {
    (forall(x: T) { p(x) implies a }) = ((exists(x: T) { p(x) }) implies a)
} by {
    if forall(x: T) { p(x) implies a } {
        if exists(x: T) { p(x) } {
            let x: T satisfy {
                p(x)
            }
            a
        }
    }
    if (exists(x: T) { p(x) }) implies a {
        forall(x: T) {
            p(x) implies a
        }
    }
}

/// Universal implication with a fixed consequent gives implication from an existential.
theorem forall_implies_const_right_forward[T](p: T -> Bool, a: Bool) {
    (forall(x: T) { p(x) implies a }) implies ((exists(x: T) { p(x) }) implies a)
} by {
    if forall(x: T) { p(x) implies a } {
        if exists(x: T) { p(x) } {
            let x: T satisfy {
                p(x)
            }
            a
        }
    }
}

/// Implication from an existential gives universal implication with a fixed consequent.
theorem forall_implies_const_right_backward[T](p: T -> Bool, a: Bool) {
    ((exists(x: T) { p(x) }) implies a) implies forall(x: T) { p(x) implies a }
} by {
    if (exists(x: T) { p(x) }) implies a {
        forall(x: T) {
            p(x) implies a
        }
    }
}

/// An existential conjunction gives the fixed left proposition and an existential.
theorem exists_and_const_left_forward[T](a: Bool, p: T -> Bool) {
    exists(x: T) { a and p(x) } implies a and exists(x: T) { p(x) }
} by {
    if exists(x: T) { a and p(x) } {
        let x: T satisfy {
            a and p(x)
        }
        exists(witness: T) {
            witness = x and p(witness)
        }
    }
}

/// A fixed left proposition and an existential give an existential conjunction.
theorem exists_and_const_left_backward[T](a: Bool, p: T -> Bool) {
    a and exists(x: T) { p(x) } implies exists(x: T) { a and p(x) }
} by {
    if a and exists(x: T) { p(x) } {
        let x: T satisfy {
            p(x)
        }
        exists(witness: T) {
            witness = x and a and p(witness)
        }
    }
}

/// Existential quantification distributes over conjunction with a fixed proposition.
theorem exists_and_const_left[T](a: Bool, p: T -> Bool) {
    (exists(x: T) { a and p(x) }) = (a and exists(x: T) { p(x) })
} by {
    exists_and_const_left_forward(a, p)
    exists_and_const_left_backward(a, p)
}

/// An existential conjunction gives an existential and the fixed right proposition.
theorem exists_and_const_right_forward[T](p: T -> Bool, a: Bool) {
    exists(x: T) { p(x) and a } implies (exists(x: T) { p(x) }) and a
} by {
    if exists(x: T) { p(x) and a } {
        let x: T satisfy {
            p(x) and a
        }
        exists(witness: T) {
            witness = x and p(witness)
        }
    }
}

/// An existential and a fixed right proposition give an existential conjunction.
theorem exists_and_const_right_backward[T](p: T -> Bool, a: Bool) {
    (exists(x: T) { p(x) }) and a implies exists(x: T) { p(x) and a }
} by {
    if (exists(x: T) { p(x) }) and a {
        let x: T satisfy {
            p(x)
        }
        exists(witness: T) {
            witness = x and p(witness) and a
        }
    }
}

/// Existential quantification distributes over conjunction with a fixed proposition on the right.
theorem exists_and_const_right[T](p: T -> Bool, a: Bool) {
    (exists(x: T) { p(x) and a }) = ((exists(x: T) { p(x) }) and a)
} by {
    exists_and_const_right_forward(p, a)
    exists_and_const_right_backward(p, a)
}

/// A rectangular universal disjunction gives a disjunction of universal statements.
theorem forall_forall_or_forward[T, U](p: T -> Bool, q: U -> Bool) {
    (forall(x: T) { forall(y: U) { p(x) or q(y) } }) implies
        ((forall(x: T) { p(x) }) or (forall(y: U) { q(y) }))
} by {
    if forall(x: T) { forall(y: U) { p(x) or q(y) } } {
        if forall(x: T) { p(x) } {
            (forall(x: T) { p(x) }) or (forall(y: U) { q(y) })
        }
        if not forall(x: T) { p(x) } {
            not_forall_imp_exists_not(p)
            let x: T satisfy {
                not p(x)
            }
            forall(y: U) {
                q(y)
            }
            (forall(x2: T) { p(x2) }) or (forall(y: U) { q(y) })
        }
    }
}

/// A disjunction of universal statements gives a rectangular universal disjunction.
theorem forall_forall_or_backward[T, U](p: T -> Bool, q: U -> Bool) {
    ((forall(x: T) { p(x) }) or (forall(y: U) { q(y) })) implies
        forall(x: T) { forall(y: U) { p(x) or q(y) } }
} by {
    if (forall(x: T) { p(x) }) or (forall(y: U) { q(y) }) {
        forall(x: T) {
            forall(y: U) {
                if forall(x2: T) { p(x2) } {
                    p(x) or q(y)
                }
                if forall(y2: U) { q(y2) } {
                    p(x) or q(y)
                }
            }
        }
    }
}

/// A rectangular universal disjunction is a disjunction of universal statements.
theorem forall_forall_or_eq_or_forall[T, U](p: T -> Bool, q: U -> Bool) {
    (forall(x: T) { forall(y: U) { p(x) or q(y) } }) =
        ((forall(x: T) { p(x) }) or (forall(y: U) { q(y) }))
} by {
    forall_forall_or_forward(p, q)
    forall_forall_or_backward(p, q)
}
