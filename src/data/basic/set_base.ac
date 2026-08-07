/// Sets with elements of type `K` are defined as Boolean functions over `K`
structure Set[K] {
    /// True if the element is in the set.
    contains: K -> Bool
}

attributes Set[K] {
    /// True if every element of `self` is an element of `s`.
    define subset(self, s: Set[K]) -> Bool {
        forall(x: K) {
            self.contains(x) implies s.contains(x)
        }
    }
}
