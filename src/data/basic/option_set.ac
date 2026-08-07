from data.basic.set import Set

/// True if an optional value lies in a set of optional values induced by a set.
define option_set_contains[T](s: Set[T], contains_none: Bool, opt: Option[T]) -> Bool {
    match opt {
        Option.none {
            contains_none
        }
        Option.some(value) {
            s.contains(value)
        }
    }
}

/// The optional set induced by a set and a truth value for `none`.
define option_set[T](s: Set[T], contains_none: Bool) -> Set[Option[T]] {
    Set[Option[T]].new(option_set_contains(s, contains_none))
}

/// The optional set containing no `none` values.
define option_some_set[T](s: Set[T]) -> Set[Option[T]] {
    option_set(s, false)
}

/// The optional set containing `none` and the `some` values from a set.
define option_with_none_set[T](s: Set[T]) -> Set[Option[T]] {
    option_set(s, true)
}

/// Membership of `some` in an optional set is membership of the contained value.
theorem option_set_contains_some[T](s: Set[T], contains_none: Bool, value: T) {
    option_set(s, contains_none).contains(Option.some(value)) = s.contains(value)
}

/// Membership of `none` in an optional set is the specified truth value.
theorem option_set_contains_none[T](s: Set[T], contains_none: Bool) {
    option_set(s, contains_none).contains(Option.none[T]) = contains_none
}

/// A contained value gives a contained `some` value.
theorem option_set_contains_some_of_contains[T](s: Set[T], contains_none: Bool, value: T) {
    s.contains(value) implies option_set(s, contains_none).contains(Option.some(value))
} by {
    if s.contains(value) {
        option_set_contains_some(s, contains_none, value)
    }
}

/// A contained `some` value gives membership of the contained value.
theorem option_set_contains_imp_contains_some[T](s: Set[T], contains_none: Bool, value: T) {
    option_set(s, contains_none).contains(Option.some(value)) implies s.contains(value)
} by {
    if option_set(s, contains_none).contains(Option.some(value)) {
        option_set_contains_some(s, contains_none, value)
    }
}

/// `none` belongs to an optional set exactly when the set is marked as containing it.
theorem option_set_contains_none_iff[T](s: Set[T], contains_none: Bool) {
    option_set(s, contains_none).contains(Option.none[T]) = contains_none
}

/// A `some` value belongs to the optional set without `none` exactly when its value belongs to the set.
theorem option_some_set_contains_some[T](s: Set[T], value: T) {
    option_some_set(s).contains(Option.some(value)) = s.contains(value)
}

/// `none` does not belong to the optional set without `none`.
theorem option_some_set_not_contains_none[T](s: Set[T]) {
    not option_some_set(s).contains(Option.none[T])
} by {
    option_set_contains_none(s, false)
}

/// A `some` value belongs to the optional set with `none` exactly when its value belongs to the set.
theorem option_with_none_set_contains_some[T](s: Set[T], value: T) {
    option_with_none_set(s).contains(Option.some(value)) = s.contains(value)
}

/// `none` belongs to the optional set with `none`.
theorem option_with_none_set_contains_none[T](s: Set[T]) {
    option_with_none_set(s).contains(Option.none[T])
} by {
    option_set_contains_none(s, true)
}

/// Mapping a set-preserving function preserves optional-set membership.
theorem option_map_preserves_option_set[T, U](source: Set[T], target: Set[U],
    contains_none: Bool, f: T -> U, opt: Option[T]) {
    option_set(source, contains_none).contains(opt) and
    forall(x: T) {
        source.contains(x) implies target.contains(f(x))
    } implies option_set(target, contains_none).contains(option_map(opt, f))
} by {
    if option_set(source, contains_none).contains(opt) and
        forall(x: T) { source.contains(x) implies target.contains(f(x)) } {
        match opt {
            Option.none {
                option_map(opt, f) = Option.none[U]
                option_set_contains_none(source, contains_none)
                contains_none
                option_set_contains_none(target, contains_none)
                option_set(target, contains_none).contains(option_map(opt, f))
            }
            Option.some(value) {
                option_map(opt, f) = Option.some(f(value))
                option_set_contains_some(source, contains_none, value)
                source.contains(value)
                target.contains(f(value))
                option_set_contains_some(target, contains_none, f(value))
                option_set(target, contains_none).contains(option_map(opt, f))
            }
        }
    }
}

