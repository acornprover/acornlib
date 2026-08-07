from list import List
from data.basic.functions import Inhabited, function_extensionality, inverse_fn, is_injective_fn, is_surjective_fn,
    is_right_inverse_fn, is_bijection_fn, bijection_fn_is_injective, bijection_fn_is_surjective,
    injective_fn_eq, surjective_fn_has_preimage, right_inverse_fn_imp_surjective_fn
from data.basic.logic import forall_forall_or_eq_or_forall
from nat import Nat
from pair import Pair
from data.basic.set import Set, set_ext, subset_refl, subset_trans, subset_antisymm, subset_contains,
    empty_set_contains_eq, universal_set_contains_eq,
    union_contains_eq, intersection_contains_eq, sets_subset_union, sets_subset_contain_union,
    sets_subset_intersection, set_supset_contains_intersection, union_comm, intersection_comm,
    union_assoc, intersection_assoc, union_idemp, intersection_idemp, union_with_empty_is_self,
    intersection_with_universal_is_self, union_with_universal_is_universal,
    intersection_with_empty_is_empty, union_intersection_distrib, indexed_union,
    indexed_intersection, indexed_union_contains_eq, indexed_union_contains_of_contains,
    indexed_union_subset_of_family_subset, indexed_intersection_contains_eq,
    indexed_intersection_contains_at, indexed_intersection_contains_of_forall,
    indexed_intersection_superset_of_subset_family, indexed_union_monotone,
    indexed_intersection_monotone, indexed_union_empty_of_forall_empty,
    indexed_union_empty_imp_forall_empty, indexed_intersection_universal_of_forall_universal,
    indexed_intersection_universal_imp_forall_universal,
    indexed_union_complement_eq_intersection_complement,
    indexed_intersection_complement_eq_union_complement, list_indexed_union, list_indexed_intersection,
    list_indexed_union_contains_eq, list_indexed_union_contains_of_contains,
    list_indexed_union_subset_of_family_subset, subset_list_indexed_intersection_of_subset_family,
    list_indexed_intersection_contains_eq, list_indexed_intersection_contains_at,
    list_indexed_intersection_contains_of_forall, list_indexed_union_monotone,
    list_indexed_intersection_monotone, list_indexed_union_subset_indexed_union,
    indexed_intersection_subset_list_indexed_intersection, range_indexed_union,
    range_indexed_intersection,
    range_indexed_union_contains_eq, range_indexed_union_contains_of_lt,
    range_indexed_union_subset_of_family_subset,
    subset_range_indexed_intersection_of_subset_family,
    range_indexed_intersection_contains_eq, range_indexed_intersection_contains_at,
    range_indexed_intersection_contains_of_forall, range_indexed_union_monotone,
    range_indexed_intersection_monotone, range_indexed_union_zero,
    range_indexed_intersection_zero, range_indexed_union_suc_eq_union,
    range_indexed_intersection_suc_eq_intersection,
    range_indexed_union_complement_eq_intersection_complement,
    range_indexed_intersection_complement_eq_union_complement,
    range_indexed_union_subset_of_bound_le, range_indexed_intersection_subset_of_bound_le,
    is_increasing, is_decreasing, seq_union, seq_intersection, seq_complement,
    seq_union_contains_eq, seq_intersection_contains_eq, seq_union_set_image,
    seq_union_complement_eq_intersection_complement,
    seq_intersection_complement_eq_union_complement,
    range_indexed_union_suc_subset_of_increasing, family_subset_range_indexed_union_suc,
    range_indexed_union_suc_of_increasing, decreasing_subset_range_indexed_intersection_suc,
    range_indexed_intersection_suc_subset_family, range_indexed_intersection_suc_of_decreasing,
    range_indexed_union_subset_seq_union, seq_intersection_subset_range_indexed_intersection,
    set_image, set_image_seq, set_image_contains_witness, set_image_contains_inverse_of_bijection,
    set_inverse_contains_imp_image_contains, maps_into_set_image,
    set_image_union, set_image_intersection_subset, set_image_intersection_reverse_of_injective,
    set_image_monotone, set_image_subset_iff_subset_preimage, set_preimage,
    set_preimage_contains_eq

/// The infimum of two sets.
define set_inf[K](a: Set[K], b: Set[K]) -> Set[K] {
    a.intersection(b)
}

/// The supremum of two sets.
define set_sup[K](a: Set[K], b: Set[K]) -> Set[K] {
    a.union(b)
}

/// Membership in a set infimum is membership in both sets.
theorem set_inf_contains_eq[K](a: Set[K], b: Set[K], x: K) {
    set_inf(a, b).contains(x) = (a.contains(x) and b.contains(x))
} by {
    intersection_contains_eq(a, b, x)
}

/// Membership in a set supremum is membership in at least one set.
theorem set_sup_contains_eq[K](a: Set[K], b: Set[K], x: K) {
    set_sup(a, b).contains(x) = (a.contains(x) or b.contains(x))
} by {
    union_contains_eq(a, b, x)
}

/// A set infimum is contained in its left argument.
theorem set_inf_subset_left[K](a: Set[K], b: Set[K]) {
    set_inf(a, b).subset(a)
} by {
    sets_subset_intersection(a, b)
    a.intersection(b).subset(a)
}

/// A set infimum is contained in its right argument.
theorem set_inf_subset_right[K](a: Set[K], b: Set[K]) {
    set_inf(a, b).subset(b)
} by {
    sets_subset_intersection(a, b)
    a.intersection(b).subset(b)
}

/// Every common subset is contained in the set infimum.
theorem set_subset_inf_of_subset_left_right[K](c: Set[K], a: Set[K], b: Set[K]) {
    c.subset(a) and c.subset(b) implies c.subset(set_inf(a, b))
} by {
    if c.subset(a) and c.subset(b) {
        a.superset(c)
        b.superset(c)
        set_supset_contains_intersection(a, b, c)
        a.intersection(b).superset(c)
        c.subset(a.intersection(b))
        c.subset(set_inf(a, b))
    }
}

/// Containment in a set infimum is exactly containment in both arguments.
theorem set_subset_inf_iff[K](c: Set[K], a: Set[K], b: Set[K]) {
    c.subset(set_inf(a, b)) = (c.subset(a) and c.subset(b))
} by {
    if c.subset(set_inf(a, b)) {
        set_inf_subset_left(a, b)
        subset_trans(c, set_inf(a, b), a)
        c.subset(a)
        set_inf_subset_right(a, b)
        subset_trans(c, set_inf(a, b), b)
        c.subset(b)
        c.subset(a) and c.subset(b)
    }
    if c.subset(a) and c.subset(b) {
        set_subset_inf_of_subset_left_right(c, a, b)
        c.subset(set_inf(a, b))
    }
    c.subset(set_inf(a, b)) = (c.subset(a) and c.subset(b))
}

/// The left argument is contained in a set supremum.
theorem set_subset_sup_left[K](a: Set[K], b: Set[K]) {
    a.subset(set_sup(a, b))
} by {
    sets_subset_union(a, b)
    a.subset(a.union(b))
}

/// The right argument is contained in a set supremum.
theorem set_subset_sup_right[K](a: Set[K], b: Set[K]) {
    b.subset(set_sup(a, b))
} by {
    sets_subset_union(a, b)
    b.subset(a.union(b))
}

/// A set supremum is contained in every common superset.
theorem set_sup_subset_of_subset_left_right[K](a: Set[K], b: Set[K], c: Set[K]) {
    a.subset(c) and b.subset(c) implies set_sup(a, b).subset(c)
} by {
    if a.subset(c) and b.subset(c) {
        sets_subset_contain_union(a, b, c)
        a.union(b).subset(c)
        set_sup(a, b).subset(c)
    }
}

/// A set supremum is contained in a set exactly when both arguments are contained in it.
theorem set_sup_subset_iff[K](a: Set[K], b: Set[K], c: Set[K]) {
    set_sup(a, b).subset(c) = (a.subset(c) and b.subset(c))
} by {
    if set_sup(a, b).subset(c) {
        set_subset_sup_left(a, b)
        subset_trans(a, set_sup(a, b), c)
        a.subset(c)
        set_subset_sup_right(a, b)
        subset_trans(b, set_sup(a, b), c)
        b.subset(c)
        a.subset(c) and b.subset(c)
    }
    if a.subset(c) and b.subset(c) {
        set_sup_subset_of_subset_left_right(a, b, c)
        set_sup(a, b).subset(c)
    }
    set_sup(a, b).subset(c) = (a.subset(c) and b.subset(c))
}

/// Inclusion in both arguments preserves set infimum.
theorem set_inf_monotone[K](a: Set[K], b: Set[K], c: Set[K], d: Set[K]) {
    a.subset(c) and b.subset(d) implies set_inf(a, b).subset(set_inf(c, d))
} by {
    if a.subset(c) and b.subset(d) {
        forall(x: K) {
            if set_inf(a, b).contains(x) {
                set_inf_contains_eq(a, b, x)
                a.subset(c) and a.contains(x)
                subset_contains(a, c, x)
                c.contains(x)
                b.subset(d) and b.contains(x)
                subset_contains(b, d, x)
                d.contains(x)
                set_inf_contains_eq(c, d, x)
                set_inf(c, d).contains(x)
            }
        }
    }
}

/// Inclusion in both arguments preserves set supremum.
theorem set_sup_monotone[K](a: Set[K], b: Set[K], c: Set[K], d: Set[K]) {
    a.subset(c) and b.subset(d) implies set_sup(a, b).subset(set_sup(c, d))
} by {
    if a.subset(c) and b.subset(d) {
        forall(x: K) {
            if set_sup(a, b).contains(x) {
                set_sup_contains_eq(a, b, x)
                if a.contains(x) {
                    a.subset(c) and a.contains(x)
                    subset_contains(a, c, x)
                    c.contains(x)
                    set_sup_contains_eq(c, d, x)
                    set_sup(c, d).contains(x)
                }
                if b.contains(x) {
                    b.subset(d) and b.contains(x)
                    subset_contains(b, d, x)
                    d.contains(x)
                    set_sup_contains_eq(c, d, x)
                    set_sup(c, d).contains(x)
                }
                set_sup(c, d).contains(x)
            }
        }
    }
}

/// Set infimum is commutative.
theorem set_inf_comm[K](a: Set[K], b: Set[K]) {
    set_inf(a, b) = set_inf(b, a)
} by {
    intersection_comm(a, b)
}

/// Set supremum is commutative.
theorem set_sup_comm[K](a: Set[K], b: Set[K]) {
    set_sup(a, b) = set_sup(b, a)
} by {
    union_comm(a, b)
}

/// Set infimum is associative.
theorem set_inf_assoc[K](a: Set[K], b: Set[K], c: Set[K]) {
    set_inf(a, set_inf(b, c)) = set_inf(set_inf(a, b), c)
} by {
    intersection_assoc(a, b, c)
}

/// Set supremum is associative.
theorem set_sup_assoc[K](a: Set[K], b: Set[K], c: Set[K]) {
    set_sup(a, set_sup(b, c)) = set_sup(set_sup(a, b), c)
} by {
    union_assoc(a, b, c)
}

/// Set infimum is idempotent.
theorem set_inf_idem[K](s: Set[K]) {
    set_inf(s, s) = s
} by {
    intersection_idemp(s)
}

/// Set supremum is idempotent.
theorem set_sup_idem[K](s: Set[K]) {
    set_sup(s, s) = s
} by {
    union_idemp(s)
}

/// The infimum with the universal set is the original set.
theorem set_inf_universal_right[K](s: Set[K]) {
    set_inf(s, Set[K].universal_set) = s
} by {
    intersection_with_universal_is_self(s)
}

/// The infimum with the universal set on the left is the original set.
theorem set_inf_universal_left[K](s: Set[K]) {
    set_inf(Set[K].universal_set, s) = s
} by {
    intersection_comm(Set[K].universal_set, s)
    intersection_with_universal_is_self(s)
}

/// The supremum with the empty set is the original set.
theorem set_sup_empty_right[K](s: Set[K]) {
    set_sup(s, Set[K].empty_set) = s
} by {
    union_with_empty_is_self(s)
}

/// The supremum with the empty set on the left is the original set.
theorem set_sup_empty_left[K](s: Set[K]) {
    set_sup(Set[K].empty_set, s) = s
} by {
    union_comm(Set[K].empty_set, s)
    union_with_empty_is_self(s)
}

/// The infimum with the empty set is empty.
theorem set_inf_empty_right[K](s: Set[K]) {
    set_inf(s, Set[K].empty_set) = Set[K].empty_set
} by {
    intersection_with_empty_is_empty(s)
}

/// The infimum with the empty set on the left is empty.
theorem set_inf_empty_left[K](s: Set[K]) {
    set_inf(Set[K].empty_set, s) = Set[K].empty_set
} by {
    intersection_comm(Set[K].empty_set, s)
    intersection_with_empty_is_empty(s)
}

/// The supremum with the universal set is universal.
theorem set_sup_universal_right[K](s: Set[K]) {
    set_sup(s, Set[K].universal_set) = Set[K].universal_set
} by {
    union_with_universal_is_universal(s)
}

/// The supremum with the universal set on the left is universal.
theorem set_sup_universal_left[K](s: Set[K]) {
    set_sup(Set[K].universal_set, s) = Set[K].universal_set
} by {
    union_comm(Set[K].universal_set, s)
    union_with_universal_is_universal(s)
}

/// Set infimum absorbs set supremum.
theorem set_inf_sup_absorb[K](a: Set[K], b: Set[K]) {
    set_inf(a, set_sup(a, b)) = a
} by {
    set_inf_subset_left(a, set_sup(a, b))
    set_inf(a, set_sup(a, b)).subset(a)
    set_subset_sup_left(a, b)
    set_subset_inf_of_subset_left_right(a, a, set_sup(a, b))
    a.subset(set_inf(a, set_sup(a, b)))
    subset_antisymm(set_inf(a, set_sup(a, b)), a)
}

/// Set supremum absorbs set infimum.
theorem set_sup_inf_absorb[K](a: Set[K], b: Set[K]) {
    set_sup(a, set_inf(a, b)) = a
} by {
    set_subset_sup_left(a, set_inf(a, b))
    a.subset(set_sup(a, set_inf(a, b)))
    set_inf_subset_left(a, b)
    set_sup_subset_of_subset_left_right(a, set_inf(a, b), a)
    set_sup(a, set_inf(a, b)).subset(a)
    subset_antisymm(set_sup(a, set_inf(a, b)), a)
}

/// Set infimum distributes over set supremum on the left.
theorem set_inf_sup_distrib_left[K](a: Set[K], b: Set[K], c: Set[K]) {
    set_inf(a, set_sup(b, c)) = set_sup(set_inf(a, b), set_inf(a, c))
} by {
    let u = set_inf(a, set_sup(b, c))
    let v = set_sup(set_inf(a, b), set_inf(a, c))
    forall(x: K) {
        if u.contains(x) {
            set_inf_contains_eq(a, set_sup(b, c), x)
            a.contains(x)
            set_sup(b, c).contains(x)
            set_sup_contains_eq(b, c, x)
            if b.contains(x) {
                set_inf_contains_eq(a, b, x)
                set_inf(a, b).contains(x)
                set_sup_contains_eq(set_inf(a, b), set_inf(a, c), x)
                v.contains(x)
            } else {
                c.contains(x)
                set_inf_contains_eq(a, c, x)
                set_inf(a, c).contains(x)
                set_sup_contains_eq(set_inf(a, b), set_inf(a, c), x)
                v.contains(x)
            }
        }
        if v.contains(x) {
            set_sup_contains_eq(set_inf(a, b), set_inf(a, c), x)
            if set_inf(a, b).contains(x) {
                set_inf_contains_eq(a, b, x)
                a.contains(x)
                b.contains(x)
                set_sup_contains_eq(b, c, x)
                set_sup(b, c).contains(x)
                set_inf_contains_eq(a, set_sup(b, c), x)
                u.contains(x)
            } else {
                set_inf(a, c).contains(x)
                set_inf_contains_eq(a, c, x)
                a.contains(x)
                c.contains(x)
                set_sup_contains_eq(b, c, x)
                set_sup(b, c).contains(x)
                set_inf_contains_eq(a, set_sup(b, c), x)
                u.contains(x)
            }
        }
        u.contains(x) = v.contains(x)
    }
    set_ext(u, v)
}

/// Set supremum distributes over set infimum on the left.
theorem set_sup_inf_distrib_left[K](a: Set[K], b: Set[K], c: Set[K]) {
    set_sup(a, set_inf(b, c)) = set_inf(set_sup(a, b), set_sup(a, c))
} by {
    union_intersection_distrib(a, b, c)
}

/// Image preserves binary set suprema.
theorem set_image_sup[T, U](a: Set[T], b: Set[T], f: T -> U) {
    set_image(set_sup(a, b), f) = set_sup(set_image(a, f), set_image(b, f))
} by {
    set_image_union(a, b, f)
}

/// Image of a binary set infimum is contained in the infimum of the images.
theorem set_image_inf_subset[T, U](a: Set[T], b: Set[T], f: T -> U) {
    set_image(set_inf(a, b), f).subset(set_inf(set_image(a, f), set_image(b, f)))
} by {
    set_image_intersection_subset(a, b, f)
}

/// Under injectivity, the infimum of two images is contained in the image of the infimum.
theorem set_image_inf_reverse_of_injective[T, U](a: Set[T], b: Set[T], f: T -> U) {
    is_injective_fn(f) implies
    set_inf(set_image(a, f), set_image(b, f)).subset(set_image(set_inf(a, b), f))
} by {
    if is_injective_fn(f) {
        forall(x1: T, x2: T) {
            if f(x1) = f(x2) {
                injective_fn_eq(f, x1, x2)
                x1 = x2
            }
        }
        set_image_intersection_reverse_of_injective(a, b, f)
        set_image(a, f).intersection(set_image(b, f)).subset(set_image(a.intersection(b), f))
        set_inf(set_image(a, f), set_image(b, f)).subset(set_image(set_inf(a, b), f))
    }
}

/// An injective map preserves binary set infima by image.
theorem set_image_inf_of_injective[T, U](a: Set[T], b: Set[T], f: T -> U) {
    is_injective_fn(f) implies
    set_image(set_inf(a, b), f) = set_inf(set_image(a, f), set_image(b, f))
} by {
    if is_injective_fn(f) {
        set_image_inf_subset(a, b, f)
        set_image_inf_reverse_of_injective(a, b, f)
        subset_antisymm(set_image(set_inf(a, b), f), set_inf(set_image(a, f), set_image(b, f)))
    }
}

/// A bijective map preserves binary set infima by image.
theorem set_image_inf_of_bijection[T, U](a: Set[T], b: Set[T], f: T -> U) {
    is_bijection_fn(f) implies
    set_image(set_inf(a, b), f) = set_inf(set_image(a, f), set_image(b, f))
} by {
    if is_bijection_fn(f) {
        bijection_fn_is_injective(f)
        is_injective_fn(f)
        set_image_inf_of_injective(a, b, f)
        set_image(set_inf(a, b), f) = set_inf(set_image(a, f), set_image(b, f))
    }
}

/// A set is contained in an injective image of a binary infimum exactly when it is contained in both images.
theorem set_subset_image_inf_iff_of_injective[T, U](s: Set[U], a: Set[T], b: Set[T],
    f: T -> U) {
    is_injective_fn(f) implies
    s.subset(set_image(set_inf(a, b), f)) =
        (s.subset(set_image(a, f)) and s.subset(set_image(b, f)))
} by {
    if is_injective_fn(f) {
        set_image_inf_of_injective(a, b, f)
        set_image(set_inf(a, b), f) = set_inf(set_image(a, f), set_image(b, f))
        set_subset_inf_iff(s, set_image(a, f), set_image(b, f))
        s.subset(set_image(set_inf(a, b), f)) =
            (s.subset(set_image(a, f)) and s.subset(set_image(b, f)))
    }
}

/// A set is contained in a bijective image of a binary infimum exactly when it is contained in both images.
theorem set_subset_image_inf_iff_of_bijection[T, U](s: Set[U], a: Set[T], b: Set[T],
    f: T -> U) {
    is_bijection_fn(f) implies
    s.subset(set_image(set_inf(a, b), f)) =
        (s.subset(set_image(a, f)) and s.subset(set_image(b, f)))
} by {
    if is_bijection_fn(f) {
        set_image_inf_of_bijection(a, b, f)
        set_image(set_inf(a, b), f) = set_inf(set_image(a, f), set_image(b, f))
        set_subset_inf_iff(s, set_image(a, f), set_image(b, f))
        s.subset(set_image(set_inf(a, b), f)) =
            (s.subset(set_image(a, f)) and s.subset(set_image(b, f)))
    }
}

/// The supremum of an indexed family of sets.
define set_sSup[K, I](family: I -> Set[K]) -> Set[K] {
    indexed_union(family)
}

/// The infimum of an indexed family of sets.
define set_sInf[K, I](family: I -> Set[K]) -> Set[K] {
    indexed_intersection(family)
}

/// The constant family whose members are empty.
define set_empty_family[K, I](i: I) -> Set[K] {
    Set[K].empty_set
}

/// The constant family whose members are universal.
define set_universal_family[K, I](i: I) -> Set[K] {
    Set[K].universal_set
}

/// The constant family whose members are a fixed set.
define set_constant_family[K, I](s: Set[K], i: I) -> Set[K] {
    s
}

/// The family formed by taking preimages of an indexed family.
define set_preimage_family[T, U, I](f: T -> U, family: I -> Set[U], i: I) -> Set[T] {
    set_preimage(f, family(i))
}

/// The family formed by taking images of an indexed family.
define set_image_family[T, U, I](family: I -> Set[T], f: T -> U, i: I) -> Set[U] {
    set_image(family(i), f)
}

/// The family formed by reindexing another family.
define set_reindex_family[K, I, J](family: I -> Set[K], h: J -> I, j: J) -> Set[K] {
    family(h(j))
}

/// The family formed by taking complements.
define set_complement_family[K, I](family: I -> Set[K], i: I) -> Set[K] {
    family(i).c
}

/// Membership in a binary set family at a point.
define set_binary_family_contains[K, I, J](family: (I, J) -> Set[K], x: K, i: I, j: J) -> Bool {
    family(i, j).contains(x)
}

/// The binary family whose members are empty.
define set_empty_binary_family[K, I, J](i: I, j: J) -> Set[K] {
    Set[K].empty_set
}

/// The binary family whose members are universal.
define set_universal_binary_family[K, I, J](i: I, j: J) -> Set[K] {
    Set[K].universal_set
}

/// The binary family whose members are a fixed set.
define set_constant_binary_family[K, I, J](s: Set[K], i: I, j: J) -> Set[K] {
    s
}

/// The binary family formed by taking preimages of a binary family.
define set_preimage_binary_family[T, U, I, J](f: T -> U, family: (I, J) -> Set[U],
    i: I, j: J) -> Set[T] {
    set_preimage(f, family(i, j))
}

/// The binary family formed by taking images of a binary family.
define set_image_binary_family[T, U, I, J](family: (I, J) -> Set[T], f: T -> U,
    i: I, j: J) -> Set[U] {
    set_image(family(i, j), f)
}

/// The binary family formed by taking complements.
define set_complement_binary_family[K, I, J](family: (I, J) -> Set[K],
    i: I, j: J) -> Set[K] {
    family(i, j).c
}

/// A binary set family reindexed componentwise.
define set_reindex_binary_family[K, I, J, I2, J2](family: (I, J) -> Set[K],
    h: I2 -> I, k: J2 -> J, i: I2, j: J2) -> Set[K] {
    family(h(i), k(j))
}

/// The ternary family whose members are empty.
define set_empty_ternary_family[K](i: Nat, j: Nat, k: Nat) -> Set[K] {
    Set[K].empty_set
}

/// The ternary family whose members are universal.
define set_universal_ternary_family[K](i: Nat, j: Nat, k: Nat) -> Set[K] {
    Set[K].universal_set
}

/// The ternary family whose members are a fixed set.
define set_constant_ternary_family[K](s: Set[K], i: Nat, j: Nat, k: Nat) -> Set[K] {
    s
}

/// The ternary family formed by taking preimages.
define set_preimage_ternary_family[T, U](f: T -> U,
    family: (Nat, Nat, Nat) -> Set[U], i: Nat, j: Nat, k: Nat) -> Set[T] {
    set_preimage(f, family(i, j, k))
}

/// The ternary family formed by taking images.
define set_image_ternary_family[T, U](family: (Nat, Nat, Nat) -> Set[T],
    f: T -> U, i: Nat, j: Nat, k: Nat) -> Set[U] {
    set_image(family(i, j, k), f)
}

/// The ternary family formed by taking complements.
define set_complement_ternary_family[K](family: (Nat, Nat, Nat) -> Set[K],
    i: Nat, j: Nat, k: Nat) -> Set[K] {
    family(i, j, k).c
}

/// A ternary set family reindexed componentwise.
define set_reindex_ternary_family[K](family: (Nat, Nat, Nat) -> Set[K],
    h: Nat -> Nat, k: Nat -> Nat, l: Nat -> Nat,
    i: Nat, j: Nat, m: Nat) -> Set[K] {
    family(h(i), k(j), l(m))
}

/// A binary set family viewed as a family over pairs.
define set_pair_family[K, I, J](family: (I, J) -> Set[K], p: Pair[I, J]) -> Set[K] {
    family(p.first, p.second)
}

/// A binary set family with the left index fixed.
define set_right_slice_family[K, I, J](family: (I, J) -> Set[K], i: I, j: J) -> Set[K] {
    family(i, j)
}

/// A binary set family with the right index fixed.
define set_left_slice_family[K, I, J](family: (I, J) -> Set[K], j: J, i: I) -> Set[K] {
    family(i, j)
}

/// The family obtained by taking suprema over the right index.
define set_sSup_right_family[K, I, J](family: (I, J) -> Set[K], i: I) -> Set[K] {
    set_sSup(set_right_slice_family(family, i))
}

/// The family obtained by taking suprema over the left index.
define set_sSup_left_family[K, I, J](family: (I, J) -> Set[K], j: J) -> Set[K] {
    set_sSup(set_left_slice_family(family, j))
}

/// The family obtained by taking infima over the right index.
define set_sInf_right_family[K, I, J](family: (I, J) -> Set[K], i: I) -> Set[K] {
    set_sInf(set_right_slice_family(family, i))
}

/// The family obtained by taking infima over the left index.
define set_sInf_left_family[K, I, J](family: (I, J) -> Set[K], j: J) -> Set[K] {
    set_sInf(set_left_slice_family(family, j))
}

/// Membership in the supremum of a family is membership in some member.
theorem set_sSup_contains_eq[K, I](family: I -> Set[K], x: K) {
    set_sSup(family).contains(x) = exists(i: I) {
        family(i).contains(x)
    }
} by {
    indexed_union_contains_eq(family, x)
}

/// Membership in the infimum of a family is membership in every member.
theorem set_sInf_contains_eq[K, I](family: I -> Set[K], x: K) {
    set_sInf(family).contains(x) = forall(i: I) {
        family(i).contains(x)
    }
} by {
    indexed_intersection_contains_eq(family, x)
}

/// Membership in an indexed member gives membership in the supremum of a binary family.
theorem set_sSup_pair_family_contains_of_contains[K, I, J](family: (I, J) -> Set[K], x: K) {
    (exists(i: I) {
        exists(j: J) {
            family(i, j).contains(x)
        }
    }) implies set_sSup(set_pair_family(family)).contains(x)
} by {
    if exists(i: I) { exists(j: J) { family(i, j).contains(x) } } {
        let i: I satisfy {
            exists(j: J) {
                family(i, j).contains(x)
            }
        }
        let j: J satisfy {
            family(i, j).contains(x)
        }
        set_pair_family(family, Pair.new(i, j)) = family(i, j)
        set_pair_family(family, Pair.new(i, j)).contains(x)
        set_sSup_contains_eq(set_pair_family(family), x)
        set_sSup(set_pair_family(family)).contains(x)
    }
}

/// Membership in the supremum of a binary family gives membership in an indexed member.
theorem set_sSup_pair_family_exists_of_contains[K, I, J](family: (I, J) -> Set[K], x: K) {
    set_sSup(set_pair_family(family)).contains(x) implies exists(i: I) {
        exists(j: J) {
            family(i, j).contains(x)
        }
    }
} by {
    if set_sSup(set_pair_family(family)).contains(x) {
        set_sSup_contains_eq(set_pair_family(family), x)
        let p: Pair[I, J] satisfy {
            set_pair_family(family, p).contains(x)
        }
        family(p.first, p.second).contains(x)
        exists(i: I) {
            i = p.first and exists(j: J) {
                j = p.second and family(i, j).contains(x)
            }
        }
    }
}

/// Membership in the supremum of a binary family is membership in some indexed member.
theorem set_sSup_pair_family_contains_eq[K, I, J](family: (I, J) -> Set[K], x: K) {
    set_sSup(set_pair_family(family)).contains(x) = exists(i: I) {
        exists(j: J) {
            family(i, j).contains(x)
        }
    }
} by {
    set_sSup_pair_family_contains_of_contains(family, x)
    set_sSup_pair_family_exists_of_contains(family, x)
}

/// Membership in the infimum of a binary family is membership in every indexed member.
theorem set_sInf_pair_family_contains_eq[K, I, J](family: (I, J) -> Set[K], x: K) {
    set_sInf(set_pair_family(family)).contains(x) = forall(i: I) {
        forall(j: J) {
            family(i, j).contains(x)
        }
    }
} by {
    if set_sInf(set_pair_family(family)).contains(x) {
        set_sInf_contains_eq(set_pair_family(family), x)
        forall(i: I) {
            forall(j: J) {
                set_pair_family(family, Pair.new(i, j)) = family(i, j)
                set_pair_family(family, Pair.new(i, j)).contains(x)
                family(i, j).contains(x)
            }
        }
    }
    if forall(i: I) {
        forall(j: J) {
            family(i, j).contains(x)
        }
    } {
        forall(p: Pair[I, J]) {
            family(p.first, p.second).contains(x)
            set_pair_family(family, p).contains(x)
        }
        set_sInf_contains_eq(set_pair_family(family), x)
        set_sInf(set_pair_family(family)).contains(x)
    }
}

/// Flattening a binary family agrees with first taking right-index suprema.
theorem set_sSup_pair_family_eq_sSup_right[K, I, J](family: (I, J) -> Set[K]) {
    set_sSup(set_pair_family(family)) = set_sSup(set_sSup_right_family(family))
} by {
    let u = set_sSup(set_pair_family(family))
    let v = set_sSup(set_sSup_right_family(family))
    forall(x: K) {
        if u.contains(x) {
            set_sSup_pair_family_exists_of_contains(family, x)
            let i: I satisfy {
                exists(j: J) {
                    family(i, j).contains(x)
                }
            }
            let j: J satisfy {
                family(i, j).contains(x)
            }
            set_sSup_contains_eq(set_right_slice_family(family, i), x)
            set_sSup(set_right_slice_family(family, i)).contains(x)
            set_sSup_right_family(family, i).contains(x)
            set_sSup_contains_eq(set_sSup_right_family(family), x)
            v.contains(x)
        }
        if v.contains(x) {
            set_sSup_contains_eq(set_sSup_right_family(family), x)
            let i: I satisfy {
                set_sSup_right_family(family, i).contains(x)
            }
            set_sSup_contains_eq(set_right_slice_family(family, i), x)
            let j: J satisfy {
                set_right_slice_family(family, i, j).contains(x)
            }
            family(i, j).contains(x)
            set_sSup_pair_family_contains_of_contains(family, x)
            u.contains(x)
        }
        u.contains(x) = v.contains(x)
    }
    set_ext(u, v)
}

/// Flattening a binary family agrees with first taking left-index suprema.
theorem set_sSup_pair_family_eq_sSup_left[K, I, J](family: (I, J) -> Set[K]) {
    set_sSup(set_pair_family(family)) = set_sSup(set_sSup_left_family(family))
} by {
    let u = set_sSup(set_pair_family(family))
    let v = set_sSup(set_sSup_left_family(family))
    forall(x: K) {
        if u.contains(x) {
            set_sSup_pair_family_exists_of_contains(family, x)
            let i: I satisfy {
                exists(j: J) {
                    family(i, j).contains(x)
                }
            }
            let j: J satisfy {
                family(i, j).contains(x)
            }
            set_sSup_contains_eq(set_left_slice_family(family, j), x)
            set_sSup(set_left_slice_family(family, j)).contains(x)
            set_sSup_left_family(family, j).contains(x)
            set_sSup_contains_eq(set_sSup_left_family(family), x)
            v.contains(x)
        }
        if v.contains(x) {
            set_sSup_contains_eq(set_sSup_left_family(family), x)
            let j: J satisfy {
                set_sSup_left_family(family, j).contains(x)
            }
            set_sSup_contains_eq(set_left_slice_family(family, j), x)
            let i: I satisfy {
                set_left_slice_family(family, j, i).contains(x)
            }
            family(i, j).contains(x)
            set_sSup_pair_family_contains_of_contains(family, x)
            u.contains(x)
        }
        u.contains(x) = v.contains(x)
    }
    set_ext(u, v)
}

/// Flattening a binary family agrees with first taking right-index infima.
theorem set_sInf_pair_family_eq_sInf_right[K, I, J](family: (I, J) -> Set[K]) {
    set_sInf(set_pair_family(family)) = set_sInf(set_sInf_right_family(family))
} by {
    let u = set_sInf(set_pair_family(family))
    let v = set_sInf(set_sInf_right_family(family))
    forall(x: K) {
        if u.contains(x) {
            set_sInf_pair_family_contains_eq(family, x)
            forall(i: I) {
                set_sInf_contains_eq(set_right_slice_family(family, i), x)
                set_sInf(set_right_slice_family(family, i)).contains(x)
                set_sInf_right_family(family, i).contains(x)
            }
            set_sInf_contains_eq(set_sInf_right_family(family), x)
            v.contains(x)
        }
        if v.contains(x) {
            set_sInf_contains_eq(set_sInf_right_family(family), x)
            forall(i: I) {
                set_sInf_right_family(family, i).contains(x)
                set_sInf_contains_eq(set_right_slice_family(family, i), x)
                forall(j: J) {
                    set_right_slice_family(family, i, j).contains(x)
                    family(i, j).contains(x)
                }
            }
            set_sInf_pair_family_contains_eq(family, x)
            u.contains(x)
        }
        u.contains(x) = v.contains(x)
    }
    set_ext(u, v)
}

/// Flattening a binary family agrees with first taking left-index infima.
theorem set_sInf_pair_family_eq_sInf_left[K, I, J](family: (I, J) -> Set[K]) {
    set_sInf(set_pair_family(family)) = set_sInf(set_sInf_left_family(family))
} by {
    let u = set_sInf(set_pair_family(family))
    let v = set_sInf(set_sInf_left_family(family))
    forall(x: K) {
        if u.contains(x) {
            set_sInf_pair_family_contains_eq(family, x)
            forall(j: J) {
                set_sInf_contains_eq(set_left_slice_family(family, j), x)
                set_sInf(set_left_slice_family(family, j)).contains(x)
                set_sInf_left_family(family, j).contains(x)
            }
            set_sInf_contains_eq(set_sInf_left_family(family), x)
            v.contains(x)
        }
        if v.contains(x) {
            set_sInf_contains_eq(set_sInf_left_family(family), x)
            forall(j: J) {
                set_sInf_left_family(family, j).contains(x)
                set_sInf_contains_eq(set_left_slice_family(family, j), x)
                forall(i: I) {
                    set_left_slice_family(family, j, i).contains(x)
                    family(i, j).contains(x)
                }
            }
            set_sInf_pair_family_contains_eq(family, x)
            u.contains(x)
        }
        u.contains(x) = v.contains(x)
    }
    set_ext(u, v)
}

/// The order of taking two suprema over a binary family does not matter.
theorem set_sSup_right_family_eq_sSup_left_family[K, I, J](family: (I, J) -> Set[K]) {
    set_sSup(set_sSup_right_family(family)) = set_sSup(set_sSup_left_family(family))
} by {
    set_sSup_pair_family_eq_sSup_right(family)
    set_sSup_pair_family_eq_sSup_left(family)
}

/// The order of taking two infima over a binary family does not matter.
theorem set_sInf_right_family_eq_sInf_left_family[K, I, J](family: (I, J) -> Set[K]) {
    set_sInf(set_sInf_right_family(family)) = set_sInf(set_sInf_left_family(family))
} by {
    set_sInf_pair_family_eq_sInf_right(family)
    set_sInf_pair_family_eq_sInf_left(family)
}

/// Every member of a family is contained in the supremum of that family.
theorem set_family_subset_sSup[K, I](family: I -> Set[K], i: I) {
    family(i).subset(set_sSup(family))
} by {
    forall(x: K) {
        if family(i).contains(x) {
            indexed_union_contains_of_contains(family, i, x)
            set_sSup(family).contains(x)
        }
    }
}

/// The supremum of a family is contained in every common superset.
theorem set_sSup_subset_of_family_subset[K, I](family: I -> Set[K], s: Set[K]) {
    (forall(i: I) { family(i).subset(s) }) implies set_sSup(family).subset(s)
} by {
    if forall(i: I) { family(i).subset(s) } {
        indexed_union_subset_of_family_subset(family, s)
        set_sSup(family).subset(s)
    }
}

/// The supremum of a family is contained in a set exactly when every member is contained in it.
theorem set_sSup_subset_iff[K, I](family: I -> Set[K], s: Set[K]) {
    set_sSup(family).subset(s) = forall(i: I) {
        family(i).subset(s)
    }
} by {
    if set_sSup(family).subset(s) {
        forall(i: I) {
            set_family_subset_sSup(family, i)
            subset_trans(family(i), set_sSup(family), s)
            family(i).subset(s)
        }
    }
    if forall(i: I) { family(i).subset(s) } {
        set_sSup_subset_of_family_subset(family, s)
        set_sSup(family).subset(s)
    }
    set_sSup(family).subset(s) = forall(i: I) {
        family(i).subset(s)
    }
}

/// The infimum of a family is contained in every member of the family.
theorem set_sInf_subset_family[K, I](family: I -> Set[K], i: I) {
    set_sInf(family).subset(family(i))
} by {
    forall(x: K) {
        if set_sInf(family).contains(x) {
            indexed_intersection_contains_at(family, i, x)
            family(i).contains(x)
        }
    }
}

/// Every common subset is contained in the infimum of a family.
theorem set_subset_sInf_of_subset_family[K, I](s: Set[K], family: I -> Set[K]) {
    (forall(i: I) { s.subset(family(i)) }) implies s.subset(set_sInf(family))
} by {
    if forall(i: I) { s.subset(family(i)) } {
        indexed_intersection_superset_of_subset_family(family, s)
        indexed_intersection(family).superset(s)
        s.subset(set_sInf(family))
    }
}

/// A set is contained in the infimum of a family exactly when it is contained in every member.
theorem set_subset_sInf_iff[K, I](s: Set[K], family: I -> Set[K]) {
    s.subset(set_sInf(family)) = forall(i: I) {
        s.subset(family(i))
    }
} by {
    if s.subset(set_sInf(family)) {
        forall(i: I) {
            set_sInf_subset_family(family, i)
            subset_trans(s, set_sInf(family), family(i))
            s.subset(family(i))
        }
    }
    if forall(i: I) { s.subset(family(i)) } {
        set_subset_sInf_of_subset_family(s, family)
        s.subset(set_sInf(family))
    }
    s.subset(set_sInf(family)) = forall(i: I) {
        s.subset(family(i))
    }
}

/// A right-nested binary supremum is contained in a set exactly when every member is contained in it.
theorem set_sSup_right_family_subset_iff[K, I, J](family: (I, J) -> Set[K], s: Set[K]) {
    set_sSup(set_sSup_right_family(family)).subset(s) = forall(i: I) {
        forall(j: J) {
            family(i, j).subset(s)
        }
    }
} by {
    if set_sSup(set_sSup_right_family(family)).subset(s) {
        set_sSup_subset_iff(set_sSup_right_family(family), s)
        forall(i: I) {
            set_sSup_right_family(family, i).subset(s)
            set_sSup_subset_iff(set_right_slice_family(family, i), s)
            forall(j: J) {
                set_right_slice_family(family, i, j).subset(s)
                family(i, j).subset(s)
            }
        }
    }
    if forall(i: I) {
        forall(j: J) {
            family(i, j).subset(s)
        }
    } {
        forall(i: I) {
            forall(j: J) {
                family(i, j).subset(s)
                set_right_slice_family(family, i, j).subset(s)
            }
            set_sSup_subset_iff(set_right_slice_family(family, i), s)
            set_sSup_right_family(family, i).subset(s)
        }
        set_sSup_subset_iff(set_sSup_right_family(family), s)
        set_sSup(set_sSup_right_family(family)).subset(s)
    }
}

/// A left-nested binary supremum is contained in a set exactly when every member is contained in it.
theorem set_sSup_left_family_subset_iff[K, I, J](family: (I, J) -> Set[K], s: Set[K]) {
    set_sSup(set_sSup_left_family(family)).subset(s) = forall(i: I) {
        forall(j: J) {
            family(i, j).subset(s)
        }
    }
} by {
    if set_sSup(set_sSup_left_family(family)).subset(s) {
        set_sSup_subset_iff(set_sSup_left_family(family), s)
        forall(i: I) {
            forall(j: J) {
                set_sSup_left_family(family, j).subset(s)
                set_sSup_subset_iff(set_left_slice_family(family, j), s)
                set_left_slice_family(family, j, i).subset(s)
                family(i, j).subset(s)
            }
        }
    }
    if forall(i: I) {
        forall(j: J) {
            family(i, j).subset(s)
        }
    } {
        forall(j: J) {
            forall(i: I) {
                family(i, j).subset(s)
                set_left_slice_family(family, j, i).subset(s)
            }
            set_sSup_subset_iff(set_left_slice_family(family, j), s)
            set_sSup_left_family(family, j).subset(s)
        }
        set_sSup_subset_iff(set_sSup_left_family(family), s)
        set_sSup(set_sSup_left_family(family)).subset(s)
    }
}

/// A set is contained in a right-nested binary infimum exactly when it is contained in every member.
theorem set_subset_sInf_right_family_iff[K, I, J](s: Set[K], family: (I, J) -> Set[K]) {
    s.subset(set_sInf(set_sInf_right_family(family))) = forall(i: I) {
        forall(j: J) {
            s.subset(family(i, j))
        }
    }
} by {
    if s.subset(set_sInf(set_sInf_right_family(family))) {
        set_subset_sInf_iff(s, set_sInf_right_family(family))
        forall(i: I) {
            s.subset(set_sInf_right_family(family, i))
            set_subset_sInf_iff(s, set_right_slice_family(family, i))
            forall(j: J) {
                s.subset(set_right_slice_family(family, i, j))
                s.subset(family(i, j))
            }
        }
    }
    if forall(i: I) {
        forall(j: J) {
            s.subset(family(i, j))
        }
    } {
        forall(i: I) {
            forall(j: J) {
                s.subset(family(i, j))
                s.subset(set_right_slice_family(family, i, j))
            }
            set_subset_sInf_iff(s, set_right_slice_family(family, i))
            s.subset(set_sInf_right_family(family, i))
        }
        set_subset_sInf_iff(s, set_sInf_right_family(family))
        s.subset(set_sInf(set_sInf_right_family(family)))
    }
}

/// A set is contained in a left-nested binary infimum exactly when it is contained in every member.
theorem set_subset_sInf_left_family_iff[K, I, J](s: Set[K], family: (I, J) -> Set[K]) {
    s.subset(set_sInf(set_sInf_left_family(family))) = forall(i: I) {
        forall(j: J) {
            s.subset(family(i, j))
        }
    }
} by {
    if s.subset(set_sInf(set_sInf_left_family(family))) {
        set_subset_sInf_iff(s, set_sInf_left_family(family))
        forall(i: I) {
            forall(j: J) {
                s.subset(set_sInf_left_family(family, j))
                set_subset_sInf_iff(s, set_left_slice_family(family, j))
                s.subset(set_left_slice_family(family, j, i))
                s.subset(family(i, j))
            }
        }
    }
    if forall(i: I) {
        forall(j: J) {
            s.subset(family(i, j))
        }
    } {
        forall(j: J) {
            forall(i: I) {
                s.subset(family(i, j))
                s.subset(set_left_slice_family(family, j, i))
            }
            set_subset_sInf_iff(s, set_left_slice_family(family, j))
            s.subset(set_sInf_left_family(family, j))
        }
        set_subset_sInf_iff(s, set_sInf_left_family(family))
        s.subset(set_sInf(set_sInf_left_family(family)))
    }
}

/// Pointwise inclusion of families preserves family suprema.
theorem set_sSup_monotone[K, I](a: I -> Set[K], b: I -> Set[K]) {
    (forall(i: I) { a(i).subset(b(i)) }) implies set_sSup(a).subset(set_sSup(b))
} by {
    if forall(i: I) { a(i).subset(b(i)) } {
        indexed_union_monotone(a, b)
        indexed_union(a).subset(indexed_union(b))
        set_sSup(a) = indexed_union(a)
        set_sSup(b) = indexed_union(b)
        set_sSup(a).subset(set_sSup(b))
    }
}

/// Pointwise inclusion of families preserves family infima.
theorem set_sInf_monotone[K, I](a: I -> Set[K], b: I -> Set[K]) {
    (forall(i: I) { a(i).subset(b(i)) }) implies set_sInf(a).subset(set_sInf(b))
} by {
    if forall(i: I) { a(i).subset(b(i)) } {
        indexed_intersection_monotone(a, b)
        indexed_intersection(a).subset(indexed_intersection(b))
        set_sInf(a) = indexed_intersection(a)
        set_sInf(b) = indexed_intersection(b)
        set_sInf(a).subset(set_sInf(b))
    }
}

/// Mutually pointwise-contained families have the same supremum.
theorem set_sSup_eq_of_family_subset[K, I](a: I -> Set[K], b: I -> Set[K]) {
    (forall(i: I) { a(i).subset(b(i)) }) and
    (forall(i: I) { b(i).subset(a(i)) })
    implies
    set_sSup(a) = set_sSup(b)
} by {
    if (forall(i: I) { a(i).subset(b(i)) }) and
        (forall(i: I) { b(i).subset(a(i)) }) {
        set_sSup_monotone(a, b)
        set_sSup_monotone(b, a)
        subset_antisymm(set_sSup(a), set_sSup(b))
    }
}

/// Mutually pointwise-contained families have the same infimum.
theorem set_sInf_eq_of_family_subset[K, I](a: I -> Set[K], b: I -> Set[K]) {
    (forall(i: I) { a(i).subset(b(i)) }) and
    (forall(i: I) { b(i).subset(a(i)) })
    implies
    set_sInf(a) = set_sInf(b)
} by {
    if (forall(i: I) { a(i).subset(b(i)) }) and
        (forall(i: I) { b(i).subset(a(i)) }) {
        set_sInf_monotone(a, b)
        set_sInf_monotone(b, a)
        subset_antisymm(set_sInf(a), set_sInf(b))
    }
}

/// Every member of a binary family is contained in the flattened supremum.
theorem set_pair_family_subset_sSup[K, I, J](family: (I, J) -> Set[K], i: I, j: J) {
    family(i, j).subset(set_sSup(set_pair_family(family)))
} by {
    forall(x: K) {
        if family(i, j).contains(x) {
            set_sSup_pair_family_contains_of_contains(family, x)
            set_sSup(set_pair_family(family)).contains(x)
        }
    }
}

/// The flattened supremum of a binary family is contained in every common superset.
theorem set_sSup_pair_family_subset_of_family_subset[K, I, J](family: (I, J) -> Set[K],
    s: Set[K]) {
    (forall(i: I) { forall(j: J) { family(i, j).subset(s) } }) implies
    set_sSup(set_pair_family(family)).subset(s)
} by {
    if forall(i: I) { forall(j: J) { family(i, j).subset(s) } } {
        forall(p: Pair[I, J]) {
            family(p.first, p.second).subset(s)
            set_pair_family(family, p).subset(s)
        }
        set_sSup_subset_of_family_subset(set_pair_family(family), s)
        set_sSup(set_pair_family(family)).subset(s)
    }
}

/// A flattened binary-family supremum is contained in a set exactly when every member is.
theorem set_sSup_pair_family_subset_iff[K, I, J](family: (I, J) -> Set[K], s: Set[K]) {
    set_sSup(set_pair_family(family)).subset(s) = forall(i: I) {
        forall(j: J) {
            family(i, j).subset(s)
        }
    }
} by {
    if set_sSup(set_pair_family(family)).subset(s) {
        forall(i: I) {
            forall(j: J) {
                set_pair_family_subset_sSup(family, i, j)
                subset_trans(family(i, j), set_sSup(set_pair_family(family)), s)
                family(i, j).subset(s)
            }
        }
    }
    if forall(i: I) { forall(j: J) { family(i, j).subset(s) } } {
        set_sSup_pair_family_subset_of_family_subset(family, s)
        set_sSup(set_pair_family(family)).subset(s)
    }
    set_sSup(set_pair_family(family)).subset(s) = forall(i: I) {
        forall(j: J) {
            family(i, j).subset(s)
        }
    }
}

/// The flattened infimum of a binary family is contained in every member.
theorem set_sInf_pair_family_subset_family[K, I, J](family: (I, J) -> Set[K], i: I, j: J) {
    set_sInf(set_pair_family(family)).subset(family(i, j))
} by {
    forall(x: K) {
        if set_sInf(set_pair_family(family)).contains(x) {
            set_sInf_pair_family_contains_eq(family, x)
            family(i, j).contains(x)
        }
    }
}

/// Every common subset is contained in the flattened infimum of a binary family.
theorem set_subset_sInf_pair_family_of_subset_family[K, I, J](s: Set[K],
    family: (I, J) -> Set[K]) {
    (forall(i: I) { forall(j: J) { s.subset(family(i, j)) } }) implies
    s.subset(set_sInf(set_pair_family(family)))
} by {
    if forall(i: I) { forall(j: J) { s.subset(family(i, j)) } } {
        forall(p: Pair[I, J]) {
            s.subset(family(p.first, p.second))
            s.subset(set_pair_family(family, p))
        }
        set_subset_sInf_of_subset_family(s, set_pair_family(family))
        s.subset(set_sInf(set_pair_family(family)))
    }
}

/// A set is contained in a flattened binary-family infimum exactly when it is contained in every member.
theorem set_subset_sInf_pair_family_iff[K, I, J](s: Set[K], family: (I, J) -> Set[K]) {
    s.subset(set_sInf(set_pair_family(family))) = forall(i: I) {
        forall(j: J) {
            s.subset(family(i, j))
        }
    }
} by {
    if s.subset(set_sInf(set_pair_family(family))) {
        forall(i: I) {
            forall(j: J) {
                set_sInf_pair_family_subset_family(family, i, j)
                subset_trans(s, set_sInf(set_pair_family(family)), family(i, j))
                s.subset(family(i, j))
            }
        }
    }
    if forall(i: I) { forall(j: J) { s.subset(family(i, j)) } } {
        set_subset_sInf_pair_family_of_subset_family(s, family)
        s.subset(set_sInf(set_pair_family(family)))
    }
    s.subset(set_sInf(set_pair_family(family))) = forall(i: I) {
        forall(j: J) {
            s.subset(family(i, j))
        }
    }
}

/// Pointwise inclusion of binary families preserves flattened suprema.
theorem set_sSup_pair_family_monotone[K, I, J](a: (I, J) -> Set[K],
    b: (I, J) -> Set[K]) {
    (forall(i: I) { forall(j: J) { a(i, j).subset(b(i, j)) } }) implies
    set_sSup(set_pair_family(a)).subset(set_sSup(set_pair_family(b)))
} by {
    if forall(i: I) { forall(j: J) { a(i, j).subset(b(i, j)) } } {
        forall(p: Pair[I, J]) {
            a(p.first, p.second).subset(b(p.first, p.second))
            set_pair_family(a, p).subset(set_pair_family(b, p))
        }
        set_sSup_monotone(set_pair_family(a), set_pair_family(b))
        set_sSup(set_pair_family(a)).subset(set_sSup(set_pair_family(b)))
    }
}

/// Pointwise inclusion of binary families preserves flattened infima.
theorem set_sInf_pair_family_monotone[K, I, J](a: (I, J) -> Set[K],
    b: (I, J) -> Set[K]) {
    (forall(i: I) { forall(j: J) { a(i, j).subset(b(i, j)) } }) implies
    set_sInf(set_pair_family(a)).subset(set_sInf(set_pair_family(b)))
} by {
    if forall(i: I) { forall(j: J) { a(i, j).subset(b(i, j)) } } {
        forall(p: Pair[I, J]) {
            a(p.first, p.second).subset(b(p.first, p.second))
            set_pair_family(a, p).subset(set_pair_family(b, p))
        }
        set_sInf_monotone(set_pair_family(a), set_pair_family(b))
        set_sInf(set_pair_family(a)).subset(set_sInf(set_pair_family(b)))
    }
}

/// Mutually pointwise-equivalent binary families have the same flattened supremum.
theorem set_sSup_pair_family_eq_of_family_subset[K, I, J](a: (I, J) -> Set[K],
    b: (I, J) -> Set[K]) {
    (forall(i: I) {
        forall(j: J) {
            a(i, j).subset(b(i, j)) and b(i, j).subset(a(i, j))
        }
    }) implies set_sSup(set_pair_family(a)) = set_sSup(set_pair_family(b))
} by {
    if forall(i: I) { forall(j: J) { a(i, j).subset(b(i, j)) and b(i, j).subset(a(i, j)) } } {
        forall(i: I) {
            forall(j: J) {
                a(i, j).subset(b(i, j))
            }
        }
        set_sSup_pair_family_monotone(a, b)
        set_sSup(set_pair_family(a)).subset(set_sSup(set_pair_family(b)))
        forall(i: I) {
            forall(j: J) {
                b(i, j).subset(a(i, j))
            }
        }
        set_sSup_pair_family_monotone(b, a)
        set_sSup(set_pair_family(b)).subset(set_sSup(set_pair_family(a)))
        subset_antisymm(set_sSup(set_pair_family(a)), set_sSup(set_pair_family(b)))
    }
}

/// Mutually pointwise-equivalent binary families have the same flattened infimum.
theorem set_sInf_pair_family_eq_of_family_subset[K, I, J](a: (I, J) -> Set[K],
    b: (I, J) -> Set[K]) {
    (forall(i: I) {
        forall(j: J) {
            a(i, j).subset(b(i, j)) and b(i, j).subset(a(i, j))
        }
    }) implies set_sInf(set_pair_family(a)) = set_sInf(set_pair_family(b))
} by {
    if forall(i: I) { forall(j: J) { a(i, j).subset(b(i, j)) and b(i, j).subset(a(i, j)) } } {
        forall(i: I) {
            forall(j: J) {
                a(i, j).subset(b(i, j))
            }
        }
        set_sInf_pair_family_monotone(a, b)
        set_sInf(set_pair_family(a)).subset(set_sInf(set_pair_family(b)))
        forall(i: I) {
            forall(j: J) {
                b(i, j).subset(a(i, j))
            }
        }
        set_sInf_pair_family_monotone(b, a)
        set_sInf(set_pair_family(b)).subset(set_sInf(set_pair_family(a)))
        subset_antisymm(set_sInf(set_pair_family(a)), set_sInf(set_pair_family(b)))
    }
}

/// The flattened supremum of constantly empty binary-family members is empty.
theorem set_sSup_empty_binary_family[K, I, J] {
    set_sSup(set_pair_family(set_empty_binary_family[K, I, J])) = Set[K].empty_set
} by {
    forall(i: I) {
        forall(j: J) {
            set_empty_binary_family[K, I, J](i, j).subset(Set[K].empty_set)
        }
    }
    set_sSup_pair_family_subset_of_family_subset(set_empty_binary_family[K, I, J],
        Set[K].empty_set)
    set_sSup(set_pair_family(set_empty_binary_family[K, I, J])).subset(Set[K].empty_set)
    Set[K].empty_set.subset(set_sSup(set_pair_family(set_empty_binary_family[K, I, J])))
    subset_antisymm(set_sSup(set_pair_family(set_empty_binary_family[K, I, J])),
        Set[K].empty_set)
}

/// The flattened infimum of constantly universal binary-family members is universal.
theorem set_sInf_universal_binary_family[K, I, J] {
    set_sInf(set_pair_family(set_universal_binary_family[K, I, J])) = Set[K].universal_set
} by {
    forall(i: I) {
        forall(j: J) {
            Set[K].universal_set.subset(set_universal_binary_family[K, I, J](i, j))
        }
    }
    set_subset_sInf_pair_family_of_subset_family(Set[K].universal_set,
        set_universal_binary_family[K, I, J])
    Set[K].universal_set.subset(set_sInf(set_pair_family(set_universal_binary_family[K, I, J])))
    set_sInf(set_pair_family(set_universal_binary_family[K, I, J])).subset(Set[K].universal_set)
    subset_antisymm(set_sInf(set_pair_family(set_universal_binary_family[K, I, J])),
        Set[K].universal_set)
}

/// The flattened supremum of a constant binary family is contained in the constant member.
theorem set_sSup_constant_binary_family_subset[K, I, J](s: Set[K]) {
    set_sSup(set_pair_family(set_constant_binary_family[K, I, J](s))).subset(s)
} by {
    forall(i: I) {
        forall(j: J) {
            set_constant_binary_family(s, i, j) = s
            set_constant_binary_family(s, i, j).subset(s)
        }
    }
    set_sSup_pair_family_subset_of_family_subset(set_constant_binary_family[K, I, J](s), s)
    set_sSup(set_pair_family(set_constant_binary_family[K, I, J](s))).subset(s)
}

/// A chosen constant binary-family member is contained in the flattened supremum.
theorem set_subset_sSup_constant_binary_family[K, I, J](s: Set[K], i0: I, j0: J) {
    s.subset(set_sSup(set_pair_family(set_constant_binary_family[K, I, J](s))))
} by {
    set_pair_family_subset_sSup(set_constant_binary_family[K, I, J](s), i0, j0)
    set_constant_binary_family(s, i0, j0) = s
    s.subset(set_sSup(set_pair_family(set_constant_binary_family[K, I, J](s))))
}

/// If the binary index types have witnesses, the flattened supremum of a constant family is the constant member.
theorem set_sSup_constant_binary_family[K, I, J](s: Set[K], i0: I, j0: J) {
    set_sSup(set_pair_family(set_constant_binary_family[K, I, J](s))) = s
} by {
    set_sSup_constant_binary_family_subset[K, I, J](s)
    set_subset_sSup_constant_binary_family(s, i0, j0)
    subset_antisymm(set_sSup(set_pair_family(set_constant_binary_family[K, I, J](s))), s)
}

/// The constant member is contained in the flattened infimum of a constant binary family.
theorem set_subset_sInf_constant_binary_family[K, I, J](s: Set[K]) {
    s.subset(set_sInf(set_pair_family(set_constant_binary_family[K, I, J](s))))
} by {
    forall(i: I) {
        forall(j: J) {
            set_constant_binary_family(s, i, j) = s
            s.subset(set_constant_binary_family(s, i, j))
        }
    }
    set_subset_sInf_pair_family_of_subset_family(s, set_constant_binary_family[K, I, J](s))
    s.subset(set_sInf(set_pair_family(set_constant_binary_family[K, I, J](s))))
}

/// The flattened infimum of a constant binary family is contained in a chosen constant member.
theorem set_sInf_constant_binary_family_subset[K, I, J](s: Set[K], i0: I, j0: J) {
    set_sInf(set_pair_family(set_constant_binary_family[K, I, J](s))).subset(s)
} by {
    set_sInf_pair_family_subset_family(set_constant_binary_family[K, I, J](s), i0, j0)
    set_constant_binary_family(s, i0, j0) = s
    set_sInf(set_pair_family(set_constant_binary_family[K, I, J](s))).subset(s)
}

/// If the binary index types have witnesses, the flattened infimum of a constant family is the constant member.
theorem set_sInf_constant_binary_family[K, I, J](s: Set[K], i0: I, j0: J) {
    set_sInf(set_pair_family(set_constant_binary_family[K, I, J](s))) = s
} by {
    set_sInf_constant_binary_family_subset(s, i0, j0)
    set_subset_sInf_constant_binary_family[K, I, J](s)
    subset_antisymm(set_sInf(set_pair_family(set_constant_binary_family[K, I, J](s))), s)
}

/// With binary index witnesses, the flattened supremum of constantly universal members is universal.
theorem set_sSup_universal_binary_family[K, I, J](i0: I, j0: J) {
    set_sSup(set_pair_family(set_universal_binary_family[K, I, J])) = Set[K].universal_set
} by {
    set_sSup_constant_binary_family(Set[K].universal_set, i0, j0)
    set_sSup(set_pair_family(set_constant_binary_family[K, I, J](Set[K].universal_set))) =
        Set[K].universal_set
    forall(i: I) {
        forall(j: J) {
            set_universal_binary_family[K, I, J](i, j) =
                set_constant_binary_family[K, I, J](Set[K].universal_set, i, j)
        }
    }
    set_sSup_pair_family_eq_of_family_subset(set_universal_binary_family[K, I, J],
        set_constant_binary_family[K, I, J](Set[K].universal_set))
    set_sSup(set_pair_family(set_universal_binary_family[K, I, J])) = Set[K].universal_set
}

/// With binary index witnesses, the flattened infimum of constantly empty members is empty.
theorem set_sInf_empty_binary_family[K, I, J](i0: I, j0: J) {
    set_sInf(set_pair_family(set_empty_binary_family[K, I, J])) = Set[K].empty_set
} by {
    set_sInf_constant_binary_family(Set[K].empty_set, i0, j0)
    set_sInf(set_pair_family(set_constant_binary_family[K, I, J](Set[K].empty_set))) =
        Set[K].empty_set
    forall(i: I) {
        forall(j: J) {
            set_empty_binary_family[K, I, J](i, j) =
                set_constant_binary_family[K, I, J](Set[K].empty_set, i, j)
        }
    }
    set_sInf_pair_family_eq_of_family_subset(set_empty_binary_family[K, I, J],
        set_constant_binary_family[K, I, J](Set[K].empty_set))
    set_sInf(set_pair_family(set_empty_binary_family[K, I, J])) = Set[K].empty_set
}

/// Flattening the complement binary family agrees with complementing the flattened family.
theorem set_pair_complement_binary_family_eq[K, I, J](family: (I, J) -> Set[K]) {
    set_pair_family(set_complement_binary_family(family)) =
        function(p: Pair[I, J]) { set_pair_family(family, p).c }
} by {
    let u = set_pair_family(set_complement_binary_family(family))
    let v = function(p: Pair[I, J]) { set_pair_family(family, p).c }
    forall(p: Pair[I, J]) {
        u(p) = set_complement_binary_family(family, p.first, p.second)
        set_complement_binary_family(family, p.first, p.second) =
            family(p.first, p.second).c
        set_pair_family(family, p) = family(p.first, p.second)
        v(p) = set_pair_family(family, p).c
        u(p) = v(p)
    }
    function_extensionality(u, v)
}

/// A componentwise reindexed flattened binary-family supremum is contained in the original supremum.
theorem set_sSup_reindex_binary_family_subset[K, I, J, I2, J2](
    family: (I, J) -> Set[K], h: I2 -> I, k: J2 -> J
) {
    set_sSup(set_pair_family(set_reindex_binary_family(family, h, k))).subset(
        set_sSup(set_pair_family(family)))
} by {
    forall(i: I2) {
        forall(j: J2) {
            set_pair_family_subset_sSup(family, h(i), k(j))
            family(h(i), k(j)).subset(set_sSup(set_pair_family(family)))
            set_reindex_binary_family(family, h, k, i, j) = family(h(i), k(j))
            set_reindex_binary_family(family, h, k, i, j).subset(
                set_sSup(set_pair_family(family)))
        }
    }
    set_sSup_pair_family_subset_of_family_subset(set_reindex_binary_family(family, h, k),
        set_sSup(set_pair_family(family)))
    set_sSup(set_pair_family(set_reindex_binary_family(family, h, k))).subset(
        set_sSup(set_pair_family(family)))
}

/// The original flattened binary-family infimum is contained in a componentwise reindexed infimum.
theorem set_sInf_subset_reindex_binary_family[K, I, J, I2, J2](
    family: (I, J) -> Set[K], h: I2 -> I, k: J2 -> J
) {
    set_sInf(set_pair_family(family)).subset(
        set_sInf(set_pair_family(set_reindex_binary_family(family, h, k))))
} by {
    forall(i: I2) {
        forall(j: J2) {
            set_sInf_pair_family_subset_family(family, h(i), k(j))
            set_sInf(set_pair_family(family)).subset(family(h(i), k(j)))
            set_reindex_binary_family(family, h, k, i, j) = family(h(i), k(j))
            set_sInf(set_pair_family(family)).subset(
                set_reindex_binary_family(family, h, k, i, j))
        }
    }
    set_subset_sInf_pair_family_of_subset_family(set_sInf(set_pair_family(family)),
        set_reindex_binary_family(family, h, k))
    set_sInf(set_pair_family(family)).subset(
        set_sInf(set_pair_family(set_reindex_binary_family(family, h, k))))
}

/// Componentwise sections give the reverse supremum inclusion for binary-family reindexing.
theorem set_sSup_subset_reindex_binary_family_of_sections[K, I, J, I2, J2](
    family: (I, J) -> Set[K], h: I2 -> I, k: J2 -> J, g: I -> I2, l: J -> J2
) {
    (forall(i: I) { h(g(i)) = i }) and (forall(j: J) { k(l(j)) = j }) implies
    set_sSup(set_pair_family(family)).subset(
        set_sSup(set_pair_family(set_reindex_binary_family(family, h, k))))
} by {
    if (forall(i: I) { h(g(i)) = i }) and (forall(j: J) { k(l(j)) = j }) {
        forall(i: I) {
            forall(j: J) {
                set_pair_family_subset_sSup(set_reindex_binary_family(family, h, k), g(i), l(j))
                set_reindex_binary_family(family, h, k, g(i), l(j)) =
                    family(h(g(i)), k(l(j)))
                h(g(i)) = i
                k(l(j)) = j
                set_reindex_binary_family(family, h, k, g(i), l(j)) = family(i, j)
                family(i, j).subset(
                    set_sSup(set_pair_family(set_reindex_binary_family(family, h, k))))
            }
        }
        set_sSup_pair_family_subset_of_family_subset(family,
            set_sSup(set_pair_family(set_reindex_binary_family(family, h, k))))
        set_sSup(set_pair_family(family)).subset(
            set_sSup(set_pair_family(set_reindex_binary_family(family, h, k))))
    }
}

/// Componentwise sections give the reverse infimum inclusion for binary-family reindexing.
theorem set_sInf_reindex_binary_family_subset_of_sections[K, I, J, I2, J2](
    family: (I, J) -> Set[K], h: I2 -> I, k: J2 -> J, g: I -> I2, l: J -> J2
) {
    (forall(i: I) { h(g(i)) = i }) and (forall(j: J) { k(l(j)) = j }) implies
    set_sInf(set_pair_family(set_reindex_binary_family(family, h, k))).subset(
        set_sInf(set_pair_family(family)))
} by {
    if (forall(i: I) { h(g(i)) = i }) and (forall(j: J) { k(l(j)) = j }) {
        forall(i: I) {
            forall(j: J) {
                set_sInf_pair_family_subset_family(set_reindex_binary_family(family, h, k),
                    g(i), l(j))
                set_sInf(set_pair_family(set_reindex_binary_family(family, h, k))).subset(
                    set_reindex_binary_family(family, h, k, g(i), l(j)))
                set_reindex_binary_family(family, h, k, g(i), l(j)) =
                    family(h(g(i)), k(l(j)))
                h(g(i)) = i
                k(l(j)) = j
                set_sInf(set_pair_family(set_reindex_binary_family(family, h, k))).subset(
                    family(i, j))
            }
        }
        set_subset_sInf_pair_family_of_subset_family(
            set_sInf(set_pair_family(set_reindex_binary_family(family, h, k))), family)
        set_sInf(set_pair_family(set_reindex_binary_family(family, h, k))).subset(
            set_sInf(set_pair_family(family)))
    }
}

/// Componentwise reindexing by maps with sections preserves flattened binary-family suprema.
theorem set_sSup_reindex_binary_family_eq_of_sections[K, I, J, I2, J2](
    family: (I, J) -> Set[K], h: I2 -> I, k: J2 -> J, g: I -> I2, l: J -> J2
) {
    (forall(i: I) { h(g(i)) = i }) and (forall(j: J) { k(l(j)) = j }) implies
    set_sSup(set_pair_family(set_reindex_binary_family(family, h, k))) =
        set_sSup(set_pair_family(family))
} by {
    if (forall(i: I) { h(g(i)) = i }) and (forall(j: J) { k(l(j)) = j }) {
        let u = set_sSup(set_pair_family(set_reindex_binary_family(family, h, k)))
        let v = set_sSup(set_pair_family(family))
        set_sSup_reindex_binary_family_subset(family, h, k)
        u.subset(v)
        set_sSup_subset_reindex_binary_family_of_sections(family, h, k, g, l)
        v.subset(u)
        u.subset(v) and v.subset(u)
        subset_antisymm(u, v)
        set_sSup(set_pair_family(set_reindex_binary_family(family, h, k))) =
            set_sSup(set_pair_family(family))
    }
}

/// Componentwise reindexing by maps with sections preserves flattened binary-family infima.
theorem set_sInf_reindex_binary_family_eq_of_sections[K, I, J, I2, J2](
    family: (I, J) -> Set[K], h: I2 -> I, k: J2 -> J, g: I -> I2, l: J -> J2
) {
    (forall(i: I) { h(g(i)) = i }) and (forall(j: J) { k(l(j)) = j }) implies
    set_sInf(set_pair_family(set_reindex_binary_family(family, h, k))) =
        set_sInf(set_pair_family(family))
} by {
    if (forall(i: I) { h(g(i)) = i }) and (forall(j: J) { k(l(j)) = j }) {
        let u = set_sInf(set_pair_family(set_reindex_binary_family(family, h, k)))
        let v = set_sInf(set_pair_family(family))
        set_sInf_subset_reindex_binary_family(family, h, k)
        v.subset(u)
        set_sInf_reindex_binary_family_subset_of_sections(family, h, k, g, l)
        u.subset(v)
        u.subset(v) and v.subset(u)
        subset_antisymm(u, v)
        set_sInf(set_pair_family(set_reindex_binary_family(family, h, k))) =
        set_sInf(set_pair_family(family))
    }
}

/// Componentwise reindexing by maps with right inverses preserves flattened binary-family suprema.
theorem set_sSup_reindex_binary_family_eq_of_right_inverses[K, I, J, I2, J2](
    family: (I, J) -> Set[K], h: I2 -> I, k: J2 -> J, g: I -> I2, l: J -> J2
) {
    is_right_inverse_fn(h, g) and is_right_inverse_fn(k, l) implies
    set_sSup(set_pair_family(set_reindex_binary_family(family, h, k))) =
        set_sSup(set_pair_family(family))
} by {
    if is_right_inverse_fn(h, g) and is_right_inverse_fn(k, l) {
        is_right_inverse_fn(h, g) = forall(i: I) {
            h(g(i)) = i
        }
        is_right_inverse_fn(k, l) = forall(j: J) {
            k(l(j)) = j
        }
        (forall(i: I) { h(g(i)) = i }) and (forall(j: J) { k(l(j)) = j })
        set_sSup_reindex_binary_family_eq_of_sections(family, h, k, g, l)
        set_sSup(set_pair_family(set_reindex_binary_family(family, h, k))) =
            set_sSup(set_pair_family(family))
    }
}

/// Componentwise reindexing by maps with right inverses preserves flattened binary-family infima.
theorem set_sInf_reindex_binary_family_eq_of_right_inverses[K, I, J, I2, J2](
    family: (I, J) -> Set[K], h: I2 -> I, k: J2 -> J, g: I -> I2, l: J -> J2
) {
    is_right_inverse_fn(h, g) and is_right_inverse_fn(k, l) implies
    set_sInf(set_pair_family(set_reindex_binary_family(family, h, k))) =
        set_sInf(set_pair_family(family))
} by {
    if is_right_inverse_fn(h, g) and is_right_inverse_fn(k, l) {
        is_right_inverse_fn(h, g) = forall(i: I) {
            h(g(i)) = i
        }
        is_right_inverse_fn(k, l) = forall(j: J) {
            k(l(j)) = j
        }
        (forall(i: I) { h(g(i)) = i }) and (forall(j: J) { k(l(j)) = j })
        set_sInf_reindex_binary_family_eq_of_sections(family, h, k, g, l)
        set_sInf(set_pair_family(set_reindex_binary_family(family, h, k))) =
            set_sInf(set_pair_family(family))
    }
}

/// Componentwise reindexing by surjective maps preserves flattened binary-family suprema.
theorem set_sSup_reindex_binary_family_eq_of_surjective[K, I, J, I2, J2](
    family: (I, J) -> Set[K], h: I2 -> I, k: J2 -> J
) {
    is_surjective_fn(h) and is_surjective_fn(k) implies
    set_sSup(set_pair_family(set_reindex_binary_family(family, h, k))) =
        set_sSup(set_pair_family(family))
} by {
    if is_surjective_fn(h) and is_surjective_fn(k) {
        let u = set_sSup(set_pair_family(set_reindex_binary_family(family, h, k)))
        let v = set_sSup(set_pair_family(family))
        set_sSup_reindex_binary_family_subset(family, h, k)
        u.subset(v)
        forall(i: I) {
            surjective_fn_has_preimage(h, i)
            let i2: I2 satisfy {
                h(i2) = i
            }
            forall(j: J) {
                surjective_fn_has_preimage(k, j)
                let j2: J2 satisfy {
                    k(j2) = j
                }
                set_pair_family_subset_sSup(set_reindex_binary_family(family, h, k), i2, j2)
                set_reindex_binary_family(family, h, k, i2, j2) = family(h(i2), k(j2))
                h(i2) = i
                k(j2) = j
                set_reindex_binary_family(family, h, k, i2, j2) = family(i, j)
                family(i, j).subset(u)
            }
        }
        set_sSup_pair_family_subset_of_family_subset(family, u)
        v.subset(u)
        u.subset(v) and v.subset(u)
        subset_antisymm(u, v)
        set_sSup(set_pair_family(set_reindex_binary_family(family, h, k))) =
            set_sSup(set_pair_family(family))
    }
}

/// Componentwise reindexing by surjective maps preserves flattened binary-family infima.
theorem set_sInf_reindex_binary_family_eq_of_surjective[K, I, J, I2, J2](
    family: (I, J) -> Set[K], h: I2 -> I, k: J2 -> J
) {
    is_surjective_fn(h) and is_surjective_fn(k) implies
    set_sInf(set_pair_family(set_reindex_binary_family(family, h, k))) =
        set_sInf(set_pair_family(family))
} by {
    if is_surjective_fn(h) and is_surjective_fn(k) {
        let u = set_sInf(set_pair_family(set_reindex_binary_family(family, h, k)))
        let v = set_sInf(set_pair_family(family))
        set_sInf_subset_reindex_binary_family(family, h, k)
        v.subset(u)
        forall(i: I) {
            surjective_fn_has_preimage(h, i)
            let i2: I2 satisfy {
                h(i2) = i
            }
            forall(j: J) {
                surjective_fn_has_preimage(k, j)
                let j2: J2 satisfy {
                    k(j2) = j
                }
                set_sInf_pair_family_subset_family(set_reindex_binary_family(family, h, k),
                    i2, j2)
                u.subset(set_reindex_binary_family(family, h, k, i2, j2))
                set_reindex_binary_family(family, h, k, i2, j2) = family(h(i2), k(j2))
                h(i2) = i
                k(j2) = j
                u.subset(family(i, j))
            }
        }
        set_subset_sInf_pair_family_of_subset_family(u, family)
        u.subset(v)
        u.subset(v) and v.subset(u)
        subset_antisymm(u, v)
        set_sInf(set_pair_family(set_reindex_binary_family(family, h, k))) =
            set_sInf(set_pair_family(family))
    }
}

/// Componentwise reindexing by bijective maps preserves flattened binary-family suprema.
theorem set_sSup_reindex_binary_family_eq_of_bijection[K, I, J, I2, J2](
    family: (I, J) -> Set[K], h: I2 -> I, k: J2 -> J
) {
    is_bijection_fn(h) and is_bijection_fn(k) implies
    set_sSup(set_pair_family(set_reindex_binary_family(family, h, k))) =
        set_sSup(set_pair_family(family))
} by {
    if is_bijection_fn(h) and is_bijection_fn(k) {
        bijection_fn_is_surjective(h)
        bijection_fn_is_surjective(k)
        is_surjective_fn(h) and is_surjective_fn(k)
        set_sSup_reindex_binary_family_eq_of_surjective(family, h, k)
        set_sSup(set_pair_family(set_reindex_binary_family(family, h, k))) =
            set_sSup(set_pair_family(family))
    }
}

/// Componentwise reindexing by bijective maps preserves flattened binary-family infima.
theorem set_sInf_reindex_binary_family_eq_of_bijection[K, I, J, I2, J2](
    family: (I, J) -> Set[K], h: I2 -> I, k: J2 -> J
) {
    is_bijection_fn(h) and is_bijection_fn(k) implies
    set_sInf(set_pair_family(set_reindex_binary_family(family, h, k))) =
        set_sInf(set_pair_family(family))
} by {
    if is_bijection_fn(h) and is_bijection_fn(k) {
        bijection_fn_is_surjective(h)
        bijection_fn_is_surjective(k)
        is_surjective_fn(h) and is_surjective_fn(k)
        set_sInf_reindex_binary_family_eq_of_surjective(family, h, k)
        set_sInf(set_pair_family(set_reindex_binary_family(family, h, k))) =
            set_sInf(set_pair_family(family))
    }
}

/// Preimage commutes with flattened binary-family set suprema.
theorem set_preimage_sSup_pair_family[T, U, I, J](f: T -> U, family: (I, J) -> Set[U]) {
    set_preimage(f, set_sSup(set_pair_family(family))) =
        set_sSup(set_pair_family(set_preimage_binary_family(f, family)))
} by {
    let u = set_preimage(f, set_sSup(set_pair_family(family)))
    let v = set_sSup(set_pair_family(set_preimage_binary_family(f, family)))
    forall(x: T) {
        if u.contains(x) {
            set_preimage_contains_eq(f, set_sSup(set_pair_family(family)), x)
            set_sSup(set_pair_family(family)).contains(f(x))
            set_sSup_pair_family_exists_of_contains(family, f(x))
            let i: I satisfy {
                exists(j: J) {
                    family(i, j).contains(f(x))
                }
            }
            let j: J satisfy {
                family(i, j).contains(f(x))
            }
            set_preimage_contains_eq(f, family(i, j), x)
            set_preimage(f, family(i, j)).contains(x)
            set_preimage_binary_family(f, family, i, j) = set_preimage(f, family(i, j))
            set_preimage_binary_family(f, family, i, j).contains(x)
            set_sSup_pair_family_contains_of_contains(set_preimage_binary_family(f, family), x)
            v.contains(x)
        }
        if v.contains(x) {
            set_sSup_pair_family_exists_of_contains(set_preimage_binary_family(f, family), x)
            let i: I satisfy {
                exists(j: J) {
                    set_preimage_binary_family(f, family, i, j).contains(x)
                }
            }
            let j: J satisfy {
                set_preimage_binary_family(f, family, i, j).contains(x)
            }
            set_preimage_binary_family(f, family, i, j) = set_preimage(f, family(i, j))
            set_preimage_contains_eq(f, family(i, j), x)
            family(i, j).contains(f(x))
            set_sSup_pair_family_contains_of_contains(family, f(x))
            set_sSup(set_pair_family(family)).contains(f(x))
            set_preimage_contains_eq(f, set_sSup(set_pair_family(family)), x)
            u.contains(x)
        }
        u.contains(x) = v.contains(x)
    }
    set_ext(u, v)
}

/// Preimage commutes with flattened binary-family set infima.
theorem set_preimage_sInf_pair_family[T, U, I, J](f: T -> U, family: (I, J) -> Set[U]) {
    set_preimage(f, set_sInf(set_pair_family(family))) =
        set_sInf(set_pair_family(set_preimage_binary_family(f, family)))
} by {
    let u = set_preimage(f, set_sInf(set_pair_family(family)))
    let v = set_sInf(set_pair_family(set_preimage_binary_family(f, family)))
    forall(x: T) {
        if u.contains(x) {
            set_preimage_contains_eq(f, set_sInf(set_pair_family(family)), x)
            set_sInf(set_pair_family(family)).contains(f(x))
            set_sInf_pair_family_contains_eq(family, f(x))
            forall(i: I) {
                forall(j: J) {
                    family(i, j).contains(f(x))
                    set_preimage_contains_eq(f, family(i, j), x)
                    set_preimage(f, family(i, j)).contains(x)
                    set_preimage_binary_family(f, family, i, j).contains(x)
                }
            }
            set_sInf_pair_family_contains_eq(set_preimage_binary_family(f, family), x)
            v.contains(x)
        }
        if v.contains(x) {
            set_sInf_pair_family_contains_eq(set_preimage_binary_family(f, family), x)
            forall(i: I) {
                forall(j: J) {
                    set_preimage_binary_family(f, family, i, j).contains(x)
                    set_preimage_binary_family(f, family, i, j) = set_preimage(f, family(i, j))
                    set_preimage_contains_eq(f, family(i, j), x)
                    family(i, j).contains(f(x))
                }
            }
            set_sInf_pair_family_contains_eq(family, f(x))
            set_sInf(set_pair_family(family)).contains(f(x))
            set_preimage_contains_eq(f, set_sInf(set_pair_family(family)), x)
            u.contains(x)
        }
        u.contains(x) = v.contains(x)
    }
    set_ext(u, v)
}

/// A preimage of a flattened binary-family supremum is contained in a set exactly when every member preimage is contained in it.
theorem set_preimage_sSup_pair_family_subset_iff[T, U, I, J](f: T -> U,
    family: (I, J) -> Set[U], s: Set[T]) {
    set_preimage(f, set_sSup(set_pair_family(family))).subset(s) = forall(i: I) {
        forall(j: J) {
            set_preimage_binary_family(f, family, i, j).subset(s)
        }
    }
} by {
    set_preimage_sSup_pair_family(f, family)
    set_sSup_pair_family_subset_iff(set_preimage_binary_family(f, family), s)
    set_preimage(f, set_sSup(set_pair_family(family))).subset(s) = forall(i: I) {
        forall(j: J) {
            set_preimage_binary_family(f, family, i, j).subset(s)
        }
    }
}

/// A set is contained in a preimage of a flattened binary-family infimum exactly when it is contained in every member preimage.
theorem set_subset_preimage_sInf_pair_family_iff[T, U, I, J](s: Set[T], f: T -> U,
    family: (I, J) -> Set[U]) {
    s.subset(set_preimage(f, set_sInf(set_pair_family(family)))) = forall(i: I) {
        forall(j: J) {
            s.subset(set_preimage_binary_family(f, family, i, j))
        }
    }
} by {
    set_preimage_sInf_pair_family(f, family)
    set_subset_sInf_pair_family_iff(s, set_preimage_binary_family(f, family))
    s.subset(set_preimage(f, set_sInf(set_pair_family(family)))) = forall(i: I) {
        forall(j: J) {
            s.subset(set_preimage_binary_family(f, family, i, j))
        }
    }
}

/// Image commutes with flattened binary-family set suprema.
theorem set_image_sSup_pair_family[T, U, I, J](family: (I, J) -> Set[T], f: T -> U) {
    set_image(set_sSup(set_pair_family(family)), f) =
        set_sSup(set_pair_family(set_image_binary_family(family, f)))
} by {
    let u = set_image(set_sSup(set_pair_family(family)), f)
    let v = set_sSup(set_pair_family(set_image_binary_family(family, f)))
    forall(y: U) {
        if u.contains(y) {
            set_image_contains_witness(set_sSup(set_pair_family(family)), f, y)
            let x: T satisfy {
                set_sSup(set_pair_family(family)).contains(x) and y = f(x)
            }
            set_sSup_pair_family_exists_of_contains(family, x)
            let i: I satisfy {
                exists(j: J) {
                    family(i, j).contains(x)
                }
            }
            let j: J satisfy {
                family(i, j).contains(x)
            }
            maps_into_set_image(family(i, j), f, x)
            set_image(family(i, j), f).contains(f(x))
            set_image_binary_family(family, f, i, j) = set_image(family(i, j), f)
            set_image_binary_family(family, f, i, j).contains(y)
            set_sSup_pair_family_contains_of_contains(set_image_binary_family(family, f), y)
            v.contains(y)
        }
        if v.contains(y) {
            set_sSup_pair_family_exists_of_contains(set_image_binary_family(family, f), y)
            let i: I satisfy {
                exists(j: J) {
                    set_image_binary_family(family, f, i, j).contains(y)
                }
            }
            let j: J satisfy {
                set_image_binary_family(family, f, i, j).contains(y)
            }
            set_image_binary_family(family, f, i, j) = set_image(family(i, j), f)
            set_image_contains_witness(family(i, j), f, y)
            let x: T satisfy {
                family(i, j).contains(x) and y = f(x)
            }
            set_sSup_pair_family_contains_of_contains(family, x)
            set_sSup(set_pair_family(family)).contains(x)
            maps_into_set_image(set_sSup(set_pair_family(family)), f, x)
            u.contains(f(x))
            u.contains(y)
        }
        u.contains(y) = v.contains(y)
    }
    set_ext(u, v)
}

/// Image of a flattened binary-family supremum is contained in a target exactly when every member maps into it.
theorem set_image_sSup_pair_family_subset_iff[T, U, I, J](family: (I, J) -> Set[T],
    s: Set[U], f: T -> U) {
    set_image(set_sSup(set_pair_family(family)), f).subset(s) = forall(i: I) {
        forall(j: J) {
            family(i, j).subset(set_preimage(f, s))
        }
    }
} by {
    set_image_subset_iff_subset_preimage(set_sSup(set_pair_family(family)), s, f)
    set_sSup_pair_family_subset_iff(family, set_preimage(f, s))
    set_image(set_sSup(set_pair_family(family)), f).subset(s) = forall(i: I) {
        forall(j: J) {
            family(i, j).subset(set_preimage(f, s))
        }
    }
}

/// The binary family formed from two indexed families by pairwise infimum.
define set_inf_product_family[K, I, J](a: I -> Set[K], b: J -> Set[K],
    i: I, j: J) -> Set[K] {
    set_inf(a(i), b(j))
}

/// The binary family formed from two indexed families by pairwise supremum.
define set_sup_product_family[K, I, J](a: I -> Set[K], b: J -> Set[K],
    i: I, j: J) -> Set[K] {
    set_sup(a(i), b(j))
}

/// Membership in a pairwise infimum product family is membership in both members.
theorem set_inf_product_family_contains_eq[K, I, J](a: I -> Set[K], b: J -> Set[K],
    i: I, j: J, x: K) {
    set_inf_product_family(a, b, i, j).contains(x) =
        (a(i).contains(x) and b(j).contains(x))
} by {
    set_inf_product_family(a, b, i, j) = set_inf(a(i), b(j))
    set_inf_contains_eq(a(i), b(j), x)
}

/// Membership in a pairwise supremum product family is membership in either member.
theorem set_sup_product_family_contains_eq[K, I, J](a: I -> Set[K], b: J -> Set[K],
    i: I, j: J, x: K) {
    set_sup_product_family(a, b, i, j).contains(x) =
        (a(i).contains(x) or b(j).contains(x))
} by {
    set_sup_product_family(a, b, i, j) = set_sup(a(i), b(j))
    set_sup_contains_eq(a(i), b(j), x)
}

/// Membership in the flattened supremum of pairwise infima gives membership in some pairwise infimum.
theorem set_sSup_inf_product_family_exists_of_contains[K, I, J](a: I -> Set[K],
    b: J -> Set[K], x: K) {
    set_sSup(set_pair_family(set_inf_product_family(a, b))).contains(x) implies
    exists(i: I) {
        exists(j: J) {
            a(i).contains(x) and b(j).contains(x)
        }
    }
} by {
    if set_sSup(set_pair_family(set_inf_product_family(a, b))).contains(x) {
        set_sSup_pair_family_exists_of_contains(set_inf_product_family(a, b), x)
        let i: I satisfy {
            exists(j: J) {
                set_inf_product_family(a, b, i, j).contains(x)
            }
        }
        let j: J satisfy {
            set_inf_product_family(a, b, i, j).contains(x)
        }
        set_inf_product_family_contains_eq(a, b, i, j, x)
        exists(i2: I) {
            i2 = i and exists(j2: J) {
                j2 = j and a(i2).contains(x) and b(j2).contains(x)
            }
        }
    }
}

/// Membership in some pairwise infimum gives membership in the flattened supremum of pairwise infima.
theorem set_sSup_inf_product_family_contains_of_exists[K, I, J](a: I -> Set[K],
    b: J -> Set[K], x: K) {
    (exists(i: I) {
        exists(j: J) {
            a(i).contains(x) and b(j).contains(x)
        }
    }) implies set_sSup(set_pair_family(set_inf_product_family(a, b))).contains(x)
} by {
    if exists(i: I) {
        exists(j: J) {
            a(i).contains(x) and b(j).contains(x)
        }
    } {
        let i: I satisfy {
            exists(j: J) {
                a(i).contains(x) and b(j).contains(x)
            }
        }
        let j: J satisfy {
            a(i).contains(x) and b(j).contains(x)
        }
        set_inf_product_family_contains_eq(a, b, i, j, x)
        set_inf_product_family(a, b, i, j).contains(x)
        set_sSup_pair_family_contains_of_contains(set_inf_product_family(a, b), x)
        set_sSup(set_pair_family(set_inf_product_family(a, b))).contains(x)
    }
}

/// Membership in the flattened supremum of pairwise infima is membership in some pairwise infimum.
theorem set_sSup_inf_product_family_contains_eq[K, I, J](a: I -> Set[K],
    b: J -> Set[K], x: K) {
    set_sSup(set_pair_family(set_inf_product_family(a, b))).contains(x) =
    exists(i: I) {
        exists(j: J) {
            a(i).contains(x) and b(j).contains(x)
        }
    }
} by {
    set_sSup_inf_product_family_exists_of_contains(a, b, x)
    set_sSup_inf_product_family_contains_of_exists(a, b, x)
    set_sSup(set_pair_family(set_inf_product_family(a, b))).contains(x) =
    exists(i: I) {
        exists(j: J) {
            a(i).contains(x) and b(j).contains(x)
        }
    }
}

/// The supremum of all pairwise infima is the infimum of the two suprema.
theorem set_sSup_inf_product_family[K, I, J](a: I -> Set[K], b: J -> Set[K]) {
    set_sSup(set_pair_family(set_inf_product_family(a, b))) =
        set_inf(set_sSup(a), set_sSup(b))
} by {
    let u = set_sSup(set_pair_family(set_inf_product_family(a, b)))
    let v = set_inf(set_sSup(a), set_sSup(b))
    forall(x: K) {
        if u.contains(x) {
            set_sSup_inf_product_family_contains_eq(a, b, x)
            let i: I satisfy {
                exists(j: J) {
                    a(i).contains(x) and b(j).contains(x)
                }
            }
            let j: J satisfy {
                a(i).contains(x) and b(j).contains(x)
            }
            set_sSup_contains_eq(a, x)
            set_sSup(a).contains(x)
            set_sSup_contains_eq(b, x)
            set_sSup(b).contains(x)
            set_inf_contains_eq(set_sSup(a), set_sSup(b), x)
            v.contains(x)
        }
        if v.contains(x) {
            set_inf_contains_eq(set_sSup(a), set_sSup(b), x)
            set_sSup_contains_eq(a, x)
            let i: I satisfy {
                a(i).contains(x)
            }
            set_sSup_contains_eq(b, x)
            let j: J satisfy {
                b(j).contains(x)
            }
            set_inf_product_family_contains_eq(a, b, i, j, x)
            set_inf_product_family(a, b, i, j).contains(x)
            set_sSup_pair_family_contains_of_contains(set_inf_product_family(a, b), x)
            u.contains(x)
        }
        u.contains(x) = v.contains(x)
    }
    set_ext(u, v)
}

/// Each pairwise infimum is contained in the infimum of the two family suprema.
theorem set_inf_product_family_subset_sSup_inf[K, I, J](a: I -> Set[K],
    b: J -> Set[K], i: I, j: J) {
    set_inf_product_family(a, b, i, j).subset(set_inf(set_sSup(a), set_sSup(b)))
} by {
    set_pair_family_subset_sSup(set_inf_product_family(a, b), i, j)
    set_sSup_inf_product_family(a, b)
    set_inf_product_family(a, b, i, j).subset(
        set_sSup(set_pair_family(set_inf_product_family(a, b))))
    set_inf_product_family(a, b, i, j).subset(set_inf(set_sSup(a), set_sSup(b)))
}

/// The infimum of the two family suprema is contained in every common upper bound of the pairwise infima.
theorem set_sSup_inf_product_subset_of_family_subset[K, I, J](a: I -> Set[K],
    b: J -> Set[K], s: Set[K]) {
    (forall(i: I) {
        forall(j: J) {
            set_inf_product_family(a, b, i, j).subset(s)
        }
    }) implies set_inf(set_sSup(a), set_sSup(b)).subset(s)
} by {
    if forall(i: I) { forall(j: J) { set_inf_product_family(a, b, i, j).subset(s) } } {
        let u = set_sSup(set_pair_family(set_inf_product_family(a, b)))
        let v = set_inf(set_sSup(a), set_sSup(b))
        set_sSup_pair_family_subset_of_family_subset(set_inf_product_family(a, b), s)
        u.subset(s)
        set_sSup_inf_product_family(a, b)
        u = v
        v = u
        v.subset(s)
        set_inf(set_sSup(a), set_sSup(b)).subset(s)
    }
}

/// The infimum of the two family suprema is contained in a set exactly when every pairwise infimum is.
theorem set_sSup_inf_product_subset_iff[K, I, J](a: I -> Set[K], b: J -> Set[K],
    s: Set[K]) {
    set_inf(set_sSup(a), set_sSup(b)).subset(s) = forall(i: I) {
        forall(j: J) {
            set_inf_product_family(a, b, i, j).subset(s)
        }
    }
} by {
    if set_inf(set_sSup(a), set_sSup(b)).subset(s) {
        forall(i: I) {
            forall(j: J) {
                set_inf_product_family_subset_sSup_inf(a, b, i, j)
                subset_trans(set_inf_product_family(a, b, i, j),
                    set_inf(set_sSup(a), set_sSup(b)), s)
                set_inf_product_family(a, b, i, j).subset(s)
            }
        }
    }
    if forall(i: I) { forall(j: J) { set_inf_product_family(a, b, i, j).subset(s) } } {
        set_sSup_inf_product_subset_of_family_subset(a, b, s)
        set_inf(set_sSup(a), set_sSup(b)).subset(s)
    }
    set_inf(set_sSup(a), set_sSup(b)).subset(s) = forall(i: I) {
        forall(j: J) {
            set_inf_product_family(a, b, i, j).subset(s)
        }
    }
}

/// The infimum of the two family suprema is the least upper bound of the pairwise infima.
theorem set_sSup_inf_product_family_is_lub[K, I, J](a: I -> Set[K], b: J -> Set[K],
    s: Set[K]) {
    (forall(i: I) {
        forall(j: J) {
            set_inf_product_family(a, b, i, j).subset(set_inf(set_sSup(a), set_sSup(b)))
        }
    }) and
    (set_inf(set_sSup(a), set_sSup(b)).subset(s) = forall(i: I) {
        forall(j: J) {
            set_inf_product_family(a, b, i, j).subset(s)
        }
    })
} by {
    forall(i: I) {
        forall(j: J) {
            set_inf_product_family_subset_sSup_inf(a, b, i, j)
        }
    }
    set_sSup_inf_product_subset_iff(a, b, s)
}

/// Pointwise inclusion in both factors preserves each pairwise infimum.
theorem set_inf_product_family_monotone[K, I, J](a: I -> Set[K], b: J -> Set[K],
    c: I -> Set[K], d: J -> Set[K], i: I, j: J) {
    a(i).subset(c(i)) and b(j).subset(d(j)) implies
    set_inf_product_family(a, b, i, j).subset(set_inf_product_family(c, d, i, j))
} by {
    if a(i).subset(c(i)) and b(j).subset(d(j)) {
        forall(x: K) {
            if set_inf_product_family(a, b, i, j).contains(x) {
                set_inf_product_family_contains_eq(a, b, i, j, x)
                c(i).contains(x)
                d(j).contains(x)
                set_inf_product_family_contains_eq(c, d, i, j, x)
                set_inf_product_family(c, d, i, j).contains(x)
            }
        }
    }
}

/// Pointwise inclusion in both factors preserves suprema of pairwise infima.
theorem set_sSup_inf_product_family_monotone[K, I, J](a: I -> Set[K], b: J -> Set[K],
    c: I -> Set[K], d: J -> Set[K]) {
    (forall(i: I) { a(i).subset(c(i)) }) and
    (forall(j: J) { b(j).subset(d(j)) }) implies
    set_sSup(set_pair_family(set_inf_product_family(a, b))).subset(
        set_sSup(set_pair_family(set_inf_product_family(c, d))))
} by {
    if (forall(i: I) { a(i).subset(c(i)) }) and (forall(j: J) { b(j).subset(d(j)) }) {
        forall(i: I) {
            forall(j: J) {
                a(i).subset(c(i))
                b(j).subset(d(j))
                set_inf_product_family_monotone(a, b, c, d, i, j)
            }
        }
        set_sSup_pair_family_monotone(set_inf_product_family(a, b),
            set_inf_product_family(c, d))
        set_sSup(set_pair_family(set_inf_product_family(a, b))).subset(
            set_sSup(set_pair_family(set_inf_product_family(c, d))))
    }
}

/// Membership in the flattened infimum of pairwise suprema gives membership in the infimum of one factor.
theorem set_sInf_sup_product_family_contains_forward[K, I, J](a: I -> Set[K],
    b: J -> Set[K], x: K) {
    set_sInf(set_pair_family(set_sup_product_family(a, b))).contains(x) implies
        (set_sInf(a).contains(x) or set_sInf(b).contains(x))
} by {
    if set_sInf(set_pair_family(set_sup_product_family(a, b))).contains(x) {
        set_sInf_pair_family_contains_eq(set_sup_product_family(a, b), x)
        forall(i: I) {
            forall(j: J) {
                set_sup_product_family(a, b, i, j) = set_sup(a(i), b(j))
                set_sup(a(i), b(j)).contains(x)
                set_sup_contains_eq(a(i), b(j), x)
                a(i).contains(x) or b(j).contains(x)
            }
        }
        forall_forall_or_eq_or_forall(function(i: I) { a(i).contains(x) },
            function(j: J) { b(j).contains(x) })
        (forall(i: I) { a(i).contains(x) }) or (forall(j: J) { b(j).contains(x) })
        set_sInf_contains_eq(a, x)
        set_sInf_contains_eq(b, x)
        if forall(i: I) { a(i).contains(x) } {
            set_sInf(a).contains(x)
            set_sInf(a).contains(x) or set_sInf(b).contains(x)
        }
        if forall(j: J) { b(j).contains(x) } {
            set_sInf(b).contains(x)
            set_sInf(a).contains(x) or set_sInf(b).contains(x)
        }
        set_sInf(a).contains(x) or set_sInf(b).contains(x)
    }
}

/// Membership in the infimum of one factor gives membership in the flattened infimum of pairwise suprema.
theorem set_sInf_sup_product_family_contains_backward[K, I, J](a: I -> Set[K],
    b: J -> Set[K], x: K) {
    (set_sInf(a).contains(x) or set_sInf(b).contains(x)) implies
        set_sInf(set_pair_family(set_sup_product_family(a, b))).contains(x)
} by {
    if set_sInf(a).contains(x) or set_sInf(b).contains(x) {
        set_sInf_contains_eq(a, x)
        set_sInf_contains_eq(b, x)
        forall_forall_or_eq_or_forall(function(i: I) { a(i).contains(x) },
            function(j: J) { b(j).contains(x) })
        forall(i: I) {
            forall(j: J) {
                if set_sInf(a).contains(x) {
                    a(i).contains(x)
                    a(i).contains(x) or b(j).contains(x)
                }
                if set_sInf(b).contains(x) {
                    b(j).contains(x)
                    a(i).contains(x) or b(j).contains(x)
                }
                a(i).contains(x) or b(j).contains(x)
                set_sup(a(i), b(j)).contains(x)
                set_sup_product_family(a, b, i, j) = set_sup(a(i), b(j))
                set_sup_product_family(a, b, i, j).contains(x)
            }
        }
        set_sInf_pair_family_contains_eq(set_sup_product_family(a, b), x)
        set_sInf(set_pair_family(set_sup_product_family(a, b))).contains(x)
    }
}

/// Membership in the flattened infimum of pairwise suprema is membership in the infimum of one factor.
theorem set_sInf_sup_product_family_contains_eq[K, I, J](a: I -> Set[K],
    b: J -> Set[K], x: K) {
    set_sInf(set_pair_family(set_sup_product_family(a, b))).contains(x) =
        (set_sInf(a).contains(x) or set_sInf(b).contains(x))
} by {
    if set_sInf(set_pair_family(set_sup_product_family(a, b))).contains(x) {
        set_sInf_sup_product_family_contains_forward(a, b, x)
        set_sInf(a).contains(x) or set_sInf(b).contains(x)
    }
    if set_sInf(a).contains(x) or set_sInf(b).contains(x) {
        set_sInf_sup_product_family_contains_backward(a, b, x)
        set_sInf(set_pair_family(set_sup_product_family(a, b))).contains(x)
    }
}

/// The infimum of all pairwise suprema is the supremum of the two infima.
theorem set_sInf_sup_product_family[K, I, J](a: I -> Set[K], b: J -> Set[K]) {
    set_sInf(set_pair_family(set_sup_product_family(a, b))) =
        set_sup(set_sInf(a), set_sInf(b))
} by {
    let u = set_sInf(set_pair_family(set_sup_product_family(a, b)))
    let v = set_sup(set_sInf(a), set_sInf(b))
    forall(x: K) {
        set_sInf_sup_product_family_contains_eq(a, b, x)
        u.contains(x) = v.contains(x)
    }
    set_ext(u, v)
}

/// The supremum of the two family infima is contained in each pairwise supremum.
theorem set_sInf_sup_subset_product_family[K, I, J](a: I -> Set[K],
    b: J -> Set[K], i: I, j: J) {
    set_sup(set_sInf(a), set_sInf(b)).subset(set_sup_product_family(a, b, i, j))
} by {
    set_sInf_pair_family_subset_family(set_sup_product_family(a, b), i, j)
    set_sInf_sup_product_family(a, b)
    set_sInf(set_pair_family(set_sup_product_family(a, b))).subset(
        set_sup_product_family(a, b, i, j))
    set_sup(set_sInf(a), set_sInf(b)).subset(set_sup_product_family(a, b, i, j))
}

/// Every common lower bound of the pairwise suprema is contained in the supremum of the two family infima.
theorem set_subset_sInf_sup_product_of_subset_family[K, I, J](s: Set[K],
    a: I -> Set[K], b: J -> Set[K]) {
    (forall(i: I) {
        forall(j: J) {
            s.subset(set_sup_product_family(a, b, i, j))
        }
    }) implies s.subset(set_sup(set_sInf(a), set_sInf(b)))
} by {
    if forall(i: I) { forall(j: J) { s.subset(set_sup_product_family(a, b, i, j)) } } {
        let u = set_sInf(set_pair_family(set_sup_product_family(a, b)))
        let v = set_sup(set_sInf(a), set_sInf(b))
        forall(p: Pair[I, J]) {
            s.subset(set_sup_product_family(a, b, p.first, p.second))
            set_pair_family(set_sup_product_family(a, b), p) =
                set_sup_product_family(a, b, p.first, p.second)
            s.subset(set_pair_family(set_sup_product_family(a, b), p))
        }
        set_subset_sInf_of_subset_family(s, set_pair_family(set_sup_product_family(a, b)))
        s.subset(set_sInf(set_pair_family(set_sup_product_family(a, b))))
        set_subset_sInf_pair_family_of_subset_family(s, set_sup_product_family(a, b))
        s.subset(u)
        set_sInf_sup_product_family(a, b)
        u = v
        s.subset(v)
        s.subset(set_sup(set_sInf(a), set_sInf(b)))
    }
}

/// A set is contained in the supremum of the two family infima exactly when it is below every pairwise supremum.
theorem set_subset_sInf_sup_product_iff[K, I, J](s: Set[K],
    a: I -> Set[K], b: J -> Set[K]) {
    s.subset(set_sup(set_sInf(a), set_sInf(b))) = forall(i: I) {
        forall(j: J) {
            s.subset(set_sup_product_family(a, b, i, j))
        }
    }
} by {
    if s.subset(set_sup(set_sInf(a), set_sInf(b))) {
        forall(i: I) {
            forall(j: J) {
                set_sInf_sup_subset_product_family(a, b, i, j)
                subset_trans(s, set_sup(set_sInf(a), set_sInf(b)),
                    set_sup_product_family(a, b, i, j))
                s.subset(set_sup_product_family(a, b, i, j))
            }
        }
    }
    if forall(i: I) { forall(j: J) { s.subset(set_sup_product_family(a, b, i, j)) } } {
        set_subset_sInf_sup_product_of_subset_family(s, a, b)
        s.subset(set_sup(set_sInf(a), set_sInf(b)))
    }
    s.subset(set_sup(set_sInf(a), set_sInf(b))) = forall(i: I) {
        forall(j: J) {
            s.subset(set_sup_product_family(a, b, i, j))
        }
    }
}

/// The supremum of the two family infima is the greatest lower bound of the pairwise suprema.
theorem set_sInf_sup_product_family_is_glb[K, I, J](s: Set[K],
    a: I -> Set[K], b: J -> Set[K]) {
    (forall(i: I) {
        forall(j: J) {
            set_sup(set_sInf(a), set_sInf(b)).subset(set_sup_product_family(a, b, i, j))
        }
    }) and
    (s.subset(set_sup(set_sInf(a), set_sInf(b))) = forall(i: I) {
        forall(j: J) {
            s.subset(set_sup_product_family(a, b, i, j))
        }
    })
} by {
    forall(i: I) {
        forall(j: J) {
            set_sInf_sup_subset_product_family(a, b, i, j)
        }
    }
    set_subset_sInf_sup_product_iff(s, a, b)
}

/// Pointwise inclusion in both factors preserves each pairwise supremum.
theorem set_sup_product_family_monotone[K, I, J](a: I -> Set[K], b: J -> Set[K],
    c: I -> Set[K], d: J -> Set[K], i: I, j: J) {
    a(i).subset(c(i)) and b(j).subset(d(j)) implies
    set_sup_product_family(a, b, i, j).subset(set_sup_product_family(c, d, i, j))
} by {
    if a(i).subset(c(i)) and b(j).subset(d(j)) {
        forall(x: K) {
            if set_sup_product_family(a, b, i, j).contains(x) {
                set_sup_product_family_contains_eq(a, b, i, j, x)
                if a(i).contains(x) {
                    c(i).contains(x)
                    set_sup_product_family_contains_eq(c, d, i, j, x)
                    set_sup_product_family(c, d, i, j).contains(x)
                }
                if b(j).contains(x) {
                    d(j).contains(x)
                    set_sup_product_family_contains_eq(c, d, i, j, x)
                    set_sup_product_family(c, d, i, j).contains(x)
                }
                set_sup_product_family(c, d, i, j).contains(x)
            }
        }
    }
}

/// Pointwise inclusion in both factors preserves infima of pairwise suprema.
theorem set_sInf_sup_product_family_monotone[K, I, J](a: I -> Set[K], b: J -> Set[K],
    c: I -> Set[K], d: J -> Set[K]) {
    (forall(i: I) { a(i).subset(c(i)) }) and
    (forall(j: J) { b(j).subset(d(j)) }) implies
    set_sInf(set_pair_family(set_sup_product_family(a, b))).subset(
        set_sInf(set_pair_family(set_sup_product_family(c, d))))
} by {
    if (forall(i: I) { a(i).subset(c(i)) }) and (forall(j: J) { b(j).subset(d(j)) }) {
        forall(i: I) {
            forall(j: J) {
                a(i).subset(c(i))
                b(j).subset(d(j))
                set_sup_product_family_monotone(a, b, c, d, i, j)
            }
        }
        set_sInf_pair_family_monotone(set_sup_product_family(a, b),
            set_sup_product_family(c, d))
        set_sInf(set_pair_family(set_sup_product_family(a, b))).subset(
            set_sInf(set_pair_family(set_sup_product_family(c, d))))
    }
}

/// The ternary family formed by first taking pairwise infima of two families, then infima with a third family.
define set_inf3_product_family[K, I, J, L](a: I -> Set[K], b: J -> Set[K],
    c: L -> Set[K], p: Pair[I, J], l: L) -> Set[K] {
    set_inf_product_family(set_pair_family(set_inf_product_family(a, b)), c, p, l)
}

/// The ternary family formed by first taking pairwise suprema of two families, then suprema with a third family.
define set_sup3_product_family[K, I, J, L](a: I -> Set[K], b: J -> Set[K],
    c: L -> Set[K], p: Pair[I, J], l: L) -> Set[K] {
    set_sup_product_family(set_pair_family(set_sup_product_family(a, b)), c, p, l)
}

/// Membership in a ternary infimum product family is membership in all three members.
theorem set_inf3_product_family_contains_eq[K, I, J, L](a: I -> Set[K],
    b: J -> Set[K], c: L -> Set[K], p: Pair[I, J], l: L, x: K) {
    set_inf3_product_family(a, b, c, p, l).contains(x) =
        (a(p.first).contains(x) and b(p.second).contains(x) and c(l).contains(x))
} by {
    set_inf3_product_family(a, b, c, p, l) =
        set_inf_product_family(set_pair_family(set_inf_product_family(a, b)), c, p, l)
    set_inf_product_family_contains_eq(set_pair_family(set_inf_product_family(a, b)),
        c, p, l, x)
    set_pair_family(set_inf_product_family(a, b), p) =
        set_inf_product_family(a, b, p.first, p.second)
    set_inf_product_family_contains_eq(a, b, p.first, p.second, x)
}

/// Membership in a ternary supremum product family is membership in at least one member.
theorem set_sup3_product_family_contains_eq[K, I, J, L](a: I -> Set[K],
    b: J -> Set[K], c: L -> Set[K], p: Pair[I, J], l: L, x: K) {
    set_sup3_product_family(a, b, c, p, l).contains(x) =
        (a(p.first).contains(x) or b(p.second).contains(x) or c(l).contains(x))
} by {
    set_sup3_product_family(a, b, c, p, l) =
        set_sup_product_family(set_pair_family(set_sup_product_family(a, b)), c, p, l)
    set_sup_product_family_contains_eq(set_pair_family(set_sup_product_family(a, b)),
        c, p, l, x)
    set_pair_family(set_sup_product_family(a, b), p) =
        set_sup_product_family(a, b, p.first, p.second)
    set_sup_product_family_contains_eq(a, b, p.first, p.second, x)
}

/// The supremum of ternary pairwise infima is the infimum of the three suprema.
theorem set_sSup_inf3_product_family[K, I, J, L](a: I -> Set[K],
    b: J -> Set[K], c: L -> Set[K]) {
    set_sSup(set_pair_family(set_inf3_product_family(a, b, c))) =
        set_inf(set_inf(set_sSup(a), set_sSup(b)), set_sSup(c))
} by {
    let ab = set_pair_family(set_inf_product_family(a, b))
    let left = set_sSup(set_pair_family(set_inf3_product_family(a, b, c)))
    let middle = set_sSup(set_pair_family(set_inf_product_family(ab, c)))
    let right = set_inf(set_inf(set_sSup(a), set_sSup(b)), set_sSup(c))
    forall(p: Pair[I, J]) {
        forall(l: L) {
            set_inf3_product_family(a, b, c, p, l) =
                set_inf_product_family(ab, c, p, l)
        }
    }
    set_sSup_pair_family_eq_of_family_subset(set_inf3_product_family(a, b, c),
        set_inf_product_family(ab, c))
    left = middle
    set_sSup_inf_product_family(ab, c)
    middle = set_inf(set_sSup(ab), set_sSup(c))
    set_sSup_inf_product_family(a, b)
    set_sSup(ab) = set_inf(set_sSup(a), set_sSup(b))
    middle = right
    left = right
}

/// The infimum of ternary pairwise suprema is the supremum of the three infima.
theorem set_sInf_sup3_product_family[K, I, J, L](a: I -> Set[K],
    b: J -> Set[K], c: L -> Set[K]) {
    set_sInf(set_pair_family(set_sup3_product_family(a, b, c))) =
        set_sup(set_sup(set_sInf(a), set_sInf(b)), set_sInf(c))
} by {
    let ab = set_pair_family(set_sup_product_family(a, b))
    let left = set_sInf(set_pair_family(set_sup3_product_family(a, b, c)))
    let middle = set_sInf(set_pair_family(set_sup_product_family(ab, c)))
    let right = set_sup(set_sup(set_sInf(a), set_sInf(b)), set_sInf(c))
    forall(p: Pair[I, J]) {
        forall(l: L) {
            set_sup3_product_family(a, b, c, p, l) =
                set_sup_product_family(ab, c, p, l)
        }
    }
    set_sInf_pair_family_eq_of_family_subset(set_sup3_product_family(a, b, c),
        set_sup_product_family(ab, c))
    left = middle
    set_sInf_sup_product_family(ab, c)
    middle = set_sup(set_sInf(ab), set_sInf(c))
    set_sInf_sup_product_family(a, b)
    set_sInf(ab) = set_sup(set_sInf(a), set_sInf(b))
    middle = right
    left = right
}

/// Every ternary pairwise infimum is contained in the infimum of the three family suprema.
theorem set_inf3_product_family_subset_sSup_inf[K, I, J, L](a: I -> Set[K],
    b: J -> Set[K], c: L -> Set[K], p: Pair[I, J], l: L) {
    set_inf3_product_family(a, b, c, p, l).subset(
        set_inf(set_inf(set_sSup(a), set_sSup(b)), set_sSup(c)))
} by {
    set_pair_family_subset_sSup(set_inf3_product_family(a, b, c), p, l)
    set_sSup_inf3_product_family(a, b, c)
    set_inf3_product_family(a, b, c, p, l).subset(
        set_sSup(set_pair_family(set_inf3_product_family(a, b, c))))
    set_inf3_product_family(a, b, c, p, l).subset(
        set_inf(set_inf(set_sSup(a), set_sSup(b)), set_sSup(c)))
}

/// The supremum of the three family infima is contained in every ternary pairwise supremum.
theorem set_sInf_sup_subset_sup3_product_family[K, I, J, L](a: I -> Set[K],
    b: J -> Set[K], c: L -> Set[K], p: Pair[I, J], l: L) {
    set_sup(set_sup(set_sInf(a), set_sInf(b)), set_sInf(c)).subset(
        set_sup3_product_family(a, b, c, p, l))
} by {
    set_sInf_pair_family_subset_family(set_sup3_product_family(a, b, c), p, l)
    set_sInf_sup3_product_family(a, b, c)
    set_sInf(set_pair_family(set_sup3_product_family(a, b, c))).subset(
        set_sup3_product_family(a, b, c, p, l))
    set_sup(set_sup(set_sInf(a), set_sInf(b)), set_sInf(c)).subset(
        set_sup3_product_family(a, b, c, p, l))
}

/// The infimum of the three family suprema is contained in a set exactly when every ternary pairwise infimum is.
theorem set_sSup_inf3_product_subset_iff[K, I, J, L](a: I -> Set[K],
    b: J -> Set[K], c: L -> Set[K], s: Set[K]) {
    set_inf(set_inf(set_sSup(a), set_sSup(b)), set_sSup(c)).subset(s) =
        forall(p: Pair[I, J]) {
            forall(l: L) {
                set_inf3_product_family(a, b, c, p, l).subset(s)
            }
        }
} by {
    set_sSup_inf3_product_family(a, b, c)
    set_sSup_pair_family_subset_iff(set_inf3_product_family(a, b, c), s)
    set_inf(set_inf(set_sSup(a), set_sSup(b)), set_sSup(c)).subset(s) =
        forall(p: Pair[I, J]) {
            forall(l: L) {
                set_inf3_product_family(a, b, c, p, l).subset(s)
            }
        }
}

/// A set is contained in the supremum of the three family infima exactly when it is below every ternary pairwise supremum.
theorem set_subset_sInf_sup3_product_iff[K, I, J, L](s: Set[K],
    a: I -> Set[K], b: J -> Set[K], c: L -> Set[K]) {
    s.subset(set_sup(set_sup(set_sInf(a), set_sInf(b)), set_sInf(c))) =
        forall(p: Pair[I, J]) {
            forall(l: L) {
                s.subset(set_sup3_product_family(a, b, c, p, l))
            }
        }
} by {
    set_sInf_sup3_product_family(a, b, c)
    set_subset_sInf_pair_family_iff(s, set_sup3_product_family(a, b, c))
    s.subset(set_sup(set_sup(set_sInf(a), set_sInf(b)), set_sInf(c))) =
        forall(p: Pair[I, J]) {
            forall(l: L) {
                s.subset(set_sup3_product_family(a, b, c, p, l))
            }
        }
}

/// The infimum of the three family suprema is the least upper bound of ternary pairwise infima.
theorem set_sSup_inf3_product_family_is_lub[K, I, J, L](a: I -> Set[K],
    b: J -> Set[K], c: L -> Set[K], s: Set[K]) {
    (forall(p: Pair[I, J]) {
        forall(l: L) {
            set_inf3_product_family(a, b, c, p, l).subset(
                set_inf(set_inf(set_sSup(a), set_sSup(b)), set_sSup(c)))
        }
    }) and
    (set_inf(set_inf(set_sSup(a), set_sSup(b)), set_sSup(c)).subset(s) =
        forall(p: Pair[I, J]) {
            forall(l: L) {
                set_inf3_product_family(a, b, c, p, l).subset(s)
            }
        })
} by {
    forall(p: Pair[I, J]) {
        forall(l: L) {
            set_inf3_product_family_subset_sSup_inf(a, b, c, p, l)
        }
    }
    set_sSup_inf3_product_subset_iff(a, b, c, s)
}

/// The supremum of the three family infima is the greatest lower bound of ternary pairwise suprema.
theorem set_sInf_sup3_product_family_is_glb[K, I, J, L](s: Set[K],
    a: I -> Set[K], b: J -> Set[K], c: L -> Set[K]) {
    (forall(p: Pair[I, J]) {
        forall(l: L) {
            set_sup(set_sup(set_sInf(a), set_sInf(b)), set_sInf(c)).subset(
                set_sup3_product_family(a, b, c, p, l))
        }
    }) and
    (s.subset(set_sup(set_sup(set_sInf(a), set_sInf(b)), set_sInf(c))) =
        forall(p: Pair[I, J]) {
            forall(l: L) {
                s.subset(set_sup3_product_family(a, b, c, p, l))
            }
        })
} by {
    forall(p: Pair[I, J]) {
        forall(l: L) {
            set_sInf_sup_subset_sup3_product_family(a, b, c, p, l)
        }
    }
    set_subset_sInf_sup3_product_iff(s, a, b, c)
}

/// Pointwise inclusion in all three factors preserves suprema of ternary pairwise infima.
theorem set_sSup_inf3_product_family_monotone[K, I, J, L](a: I -> Set[K],
    b: J -> Set[K], c: L -> Set[K], d: I -> Set[K], e: J -> Set[K],
    f: L -> Set[K]) {
    (forall(i: I) { a(i).subset(d(i)) }) and
    (forall(j: J) { b(j).subset(e(j)) }) and
    (forall(l: L) { c(l).subset(f(l)) }) implies
    set_sSup(set_pair_family(set_inf3_product_family(a, b, c))).subset(
        set_sSup(set_pair_family(set_inf3_product_family(d, e, f))))
} by {
    if (forall(i: I) { a(i).subset(d(i)) }) and
        (forall(j: J) { b(j).subset(e(j)) }) and
        (forall(l: L) { c(l).subset(f(l)) }) {
        forall(p: Pair[I, J]) {
            forall(l: L) {
                forall(x: K) {
                    if set_inf3_product_family(a, b, c, p, l).contains(x) {
                        set_inf3_product_family_contains_eq(a, b, c, p, l, x)
                        a(p.first).contains(x)
                        b(p.second).contains(x)
                        c(l).contains(x)
                        d(p.first).contains(x)
                        e(p.second).contains(x)
                        f(l).contains(x)
                        set_inf3_product_family_contains_eq(d, e, f, p, l, x)
                        set_inf3_product_family(d, e, f, p, l).contains(x)
                        set_sSup_pair_family_contains_of_contains(set_inf3_product_family(d, e, f), x)
                        set_sSup(set_pair_family(set_inf3_product_family(d, e, f))).contains(x)
                    }
                }
                set_inf3_product_family(a, b, c, p, l).subset(
                    set_sSup(set_pair_family(set_inf3_product_family(d, e, f))))
            }
        }
        set_sSup_pair_family_subset_of_family_subset(set_inf3_product_family(a, b, c),
            set_sSup(set_pair_family(set_inf3_product_family(d, e, f))))
        set_sSup(set_pair_family(set_inf3_product_family(a, b, c))).subset(
            set_sSup(set_pair_family(set_inf3_product_family(d, e, f))))
    }
}

/// Pointwise inclusion in all three factors preserves infima of ternary pairwise suprema.
theorem set_sInf_sup3_product_family_monotone[K, I, J, L](a: I -> Set[K],
    b: J -> Set[K], c: L -> Set[K], d: I -> Set[K], e: J -> Set[K],
    f: L -> Set[K]) {
    (forall(i: I) { a(i).subset(d(i)) }) and
    (forall(j: J) { b(j).subset(e(j)) }) and
    (forall(l: L) { c(l).subset(f(l)) }) implies
    set_sInf(set_pair_family(set_sup3_product_family(a, b, c))).subset(
        set_sInf(set_pair_family(set_sup3_product_family(d, e, f))))
} by {
    if (forall(i: I) { a(i).subset(d(i)) }) and
        (forall(j: J) { b(j).subset(e(j)) }) and
        (forall(l: L) { c(l).subset(f(l)) }) {
        forall(p: Pair[I, J]) {
            forall(l: L) {
                forall(x: K) {
                    if set_sup3_product_family(a, b, c, p, l).contains(x) {
                        set_sup3_product_family_contains_eq(a, b, c, p, l, x)
                        if a(p.first).contains(x) {
                            d(p.first).contains(x)
                            set_sup3_product_family_contains_eq(d, e, f, p, l, x)
                            set_sup3_product_family(d, e, f, p, l).contains(x)
                        }
                        if b(p.second).contains(x) {
                            e(p.second).contains(x)
                            set_sup3_product_family_contains_eq(d, e, f, p, l, x)
                            set_sup3_product_family(d, e, f, p, l).contains(x)
                        }
                        if c(l).contains(x) {
                            f(l).contains(x)
                            set_sup3_product_family_contains_eq(d, e, f, p, l, x)
                            set_sup3_product_family(d, e, f, p, l).contains(x)
                        }
                        set_sup3_product_family(d, e, f, p, l).contains(x)
                    }
                }
                set_sInf_pair_family_subset_family(set_sup3_product_family(a, b, c), p, l)
                forall(x: K) {
                    if set_sInf(set_pair_family(set_sup3_product_family(a, b, c))).contains(x) {
                        set_sup3_product_family(a, b, c, p, l).contains(x)
                        set_sup3_product_family_contains_eq(a, b, c, p, l, x)
                        if a(p.first).contains(x) {
                            d(p.first).contains(x)
                            set_sup3_product_family_contains_eq(d, e, f, p, l, x)
                            set_sup3_product_family(d, e, f, p, l).contains(x)
                        }
                        if b(p.second).contains(x) {
                            e(p.second).contains(x)
                            set_sup3_product_family_contains_eq(d, e, f, p, l, x)
                            set_sup3_product_family(d, e, f, p, l).contains(x)
                        }
                        if c(l).contains(x) {
                            f(l).contains(x)
                            set_sup3_product_family_contains_eq(d, e, f, p, l, x)
                            set_sup3_product_family(d, e, f, p, l).contains(x)
                        }
                        set_sup3_product_family(d, e, f, p, l).contains(x)
                    }
                }
                set_sInf(set_pair_family(set_sup3_product_family(a, b, c))).subset(
                    set_sup3_product_family(d, e, f, p, l))
            }
        }
        set_subset_sInf_pair_family_of_subset_family(
            set_sInf(set_pair_family(set_sup3_product_family(a, b, c))),
            set_sup3_product_family(d, e, f))
        set_sInf(set_pair_family(set_sup3_product_family(a, b, c))).subset(
            set_sInf(set_pair_family(set_sup3_product_family(d, e, f))))
    }
}

/// A refinement of each member by a member of another family gives a supremum inclusion.
theorem set_sSup_subset_of_family_refines[K, I, J](a: I -> Set[K], b: J -> Set[K]) {
    (forall(i: I) { exists(j: J) { a(i).subset(b(j)) } }) implies
    set_sSup(a).subset(set_sSup(b))
} by {
    if forall(i: I) { exists(j: J) { a(i).subset(b(j)) } } {
        forall(x: K) {
            if set_sSup(a).contains(x) {
                set_sSup_contains_eq(a, x)
                let i: I satisfy {
                    a(i).contains(x)
                }
                let j: J satisfy {
                    a(i).subset(b(j))
                }
                b(j).contains(x)
                set_sSup_contains_eq(b, x)
                set_sSup(b).contains(x)
            }
        }
    }
}

/// A cofinal refinement of lower bounds gives an infimum inclusion.
theorem set_sInf_subset_of_family_refines[K, I, J](a: I -> Set[K], b: J -> Set[K]) {
    (forall(j: J) { exists(i: I) { a(i).subset(b(j)) } }) implies
    set_sInf(a).subset(set_sInf(b))
} by {
    if forall(j: J) { exists(i: I) { a(i).subset(b(j)) } } {
        forall(x: K) {
            if set_sInf(a).contains(x) {
                set_sInf_contains_eq(a, x)
                forall(j: J) {
                    let i: I satisfy {
                        a(i).subset(b(j))
                    }
                    a(i).contains(x)
                    b(j).contains(x)
                }
                set_sInf_contains_eq(b, x)
                set_sInf(b).contains(x)
            }
        }
    }
}

/// If every member of one family occurs in another, its supremum is contained in the other supremum.
theorem set_sSup_subset_of_family_occurs[K, I, J](a: I -> Set[K], b: J -> Set[K]) {
    (forall(i: I) { exists(j: J) { a(i) = b(j) } }) implies
    set_sSup(a).subset(set_sSup(b))
} by {
    if forall(i: I) { exists(j: J) { a(i) = b(j) } } {
        forall(i: I) {
            let j: J satisfy {
                a(i) = b(j)
            }
            a(i).subset(b(j))
        }
        set_sSup_subset_of_family_refines(a, b)
        set_sSup(a).subset(set_sSup(b))
    }
}

/// If every lower-bound member of one family occurs in another, the first infimum is contained in the other.
theorem set_sInf_subset_of_family_occurs[K, I, J](a: I -> Set[K], b: J -> Set[K]) {
    (forall(j: J) { exists(i: I) { a(i) = b(j) } }) implies
    set_sInf(a).subset(set_sInf(b))
} by {
    if forall(j: J) { exists(i: I) { a(i) = b(j) } } {
        forall(j: J) {
            let i: I satisfy {
                a(i) = b(j)
            }
            a(i).subset(b(j))
        }
        set_sInf_subset_of_family_refines(a, b)
        set_sInf(a).subset(set_sInf(b))
    }
}

/// Mutually cofinal family refinements give the same supremum.
theorem set_sSup_eq_of_family_refines[K, I, J](a: I -> Set[K], b: J -> Set[K]) {
    (forall(i: I) { exists(j: J) { a(i).subset(b(j)) } }) and
    (forall(j: J) { exists(i: I) { b(j).subset(a(i)) } })
    implies
    set_sSup(a) = set_sSup(b)
} by {
    if (forall(i: I) { exists(j: J) { a(i).subset(b(j)) } }) and
        (forall(j: J) { exists(i: I) { b(j).subset(a(i)) } }) {
        let u = set_sSup(a)
        let v = set_sSup(b)
        set_sSup_subset_of_family_refines(a, b)
        u.subset(v)
        set_sSup_subset_of_family_refines(b, a)
        v.subset(u)
        u.subset(v) and v.subset(u)
        subset_antisymm(u, v)
        set_sSup(a) = set_sSup(b)
    }
}

/// Mutually cofinal lower-bound refinements give the same infimum.
theorem set_sInf_eq_of_family_refines[K, I, J](a: I -> Set[K], b: J -> Set[K]) {
    (forall(j: J) { exists(i: I) { a(i).subset(b(j)) } }) and
    (forall(i: I) { exists(j: J) { b(j).subset(a(i)) } })
    implies
    set_sInf(a) = set_sInf(b)
} by {
    if (forall(j: J) { exists(i: I) { a(i).subset(b(j)) } }) and
        (forall(i: I) { exists(j: J) { b(j).subset(a(i)) } }) {
        let u = set_sInf(a)
        let v = set_sInf(b)
        set_sInf_subset_of_family_refines(a, b)
        u.subset(v)
        set_sInf_subset_of_family_refines(b, a)
        v.subset(u)
        u.subset(v) and v.subset(u)
        subset_antisymm(u, v)
        set_sInf(a) = set_sInf(b)
    }
}

/// Families with the same occurring members have the same supremum.
theorem set_sSup_eq_of_family_occurs[K, I, J](a: I -> Set[K], b: J -> Set[K]) {
    (forall(i: I) { exists(j: J) { a(i) = b(j) } }) and
    (forall(j: J) { exists(i: I) { b(j) = a(i) } })
    implies
    set_sSup(a) = set_sSup(b)
} by {
    if (forall(i: I) { exists(j: J) { a(i) = b(j) } }) and
        (forall(j: J) { exists(i: I) { b(j) = a(i) } }) {
        let u = set_sSup(a)
        let v = set_sSup(b)
        set_sSup_subset_of_family_occurs(a, b)
        u.subset(v)
        set_sSup_subset_of_family_occurs(b, a)
        v.subset(u)
        u.subset(v) and v.subset(u)
        subset_antisymm(u, v)
        set_sSup(a) = set_sSup(b)
    }
}

/// Families with the same occurring lower-bound members have the same infimum.
theorem set_sInf_eq_of_family_occurs[K, I, J](a: I -> Set[K], b: J -> Set[K]) {
    (forall(j: J) { exists(i: I) { a(i) = b(j) } }) and
    (forall(i: I) { exists(j: J) { b(j) = a(i) } })
    implies
    set_sInf(a) = set_sInf(b)
} by {
    if (forall(j: J) { exists(i: I) { a(i) = b(j) } }) and
        (forall(i: I) { exists(j: J) { b(j) = a(i) } }) {
        let u = set_sInf(a)
        let v = set_sInf(b)
        set_sInf_subset_of_family_occurs(a, b)
        u.subset(v)
        set_sInf_subset_of_family_occurs(b, a)
        v.subset(u)
        u.subset(v) and v.subset(u)
        subset_antisymm(u, v)
        set_sInf(a) = set_sInf(b)
    }
}

/// A reindexed family supremum is contained in the original supremum.
theorem set_sSup_reindex_subset[K, I, J](family: I -> Set[K], h: J -> I) {
    set_sSup(set_reindex_family(family, h)).subset(set_sSup(family))
} by {
    forall(j: J) {
        set_reindex_family(family, h, j) = family(h(j))
        set_family_subset_sSup(family, h(j))
        set_reindex_family(family, h, j).subset(set_sSup(family))
    }
    set_sSup_subset_of_family_subset(set_reindex_family(family, h), set_sSup(family))
    set_sSup(set_reindex_family(family, h)).subset(set_sSup(family))
}

/// The original infimum is contained in a reindexed family infimum.
theorem set_sInf_subset_reindex[K, I, J](family: I -> Set[K], h: J -> I) {
    set_sInf(family).subset(set_sInf(set_reindex_family(family, h)))
} by {
    forall(j: J) {
        set_sInf_subset_family(family, h(j))
        set_sInf(family).subset(family(h(j)))
        set_reindex_family(family, h, j) = family(h(j))
        set_sInf(family).subset(set_reindex_family(family, h, j))
    }
    set_subset_sInf_of_subset_family(set_sInf(family), set_reindex_family(family, h))
    set_sInf(family).subset(set_sInf(set_reindex_family(family, h)))
}

/// A reindexing map with a section gives the reverse supremum inclusion.
theorem set_sSup_subset_reindex_of_section[K, I, J](family: I -> Set[K], h: J -> I, g: I -> J) {
    (forall(i: I) { h(g(i)) = i }) implies
    set_sSup(family).subset(set_sSup(set_reindex_family(family, h)))
} by {
    if forall(i: I) { h(g(i)) = i } {
        forall(i: I) {
            set_family_subset_sSup(set_reindex_family(family, h), g(i))
            set_reindex_family(family, h, g(i)) = family(h(g(i)))
            h(g(i)) = i
            set_reindex_family(family, h, g(i)) = family(i)
            family(i).subset(set_sSup(set_reindex_family(family, h)))
        }
        set_sSup_subset_of_family_subset(family, set_sSup(set_reindex_family(family, h)))
        set_sSup(family).subset(set_sSup(set_reindex_family(family, h)))
    }
}

/// A reindexing map with a section gives the reverse infimum inclusion.
theorem set_sInf_reindex_subset_of_section[K, I, J](family: I -> Set[K], h: J -> I, g: I -> J) {
    (forall(i: I) { h(g(i)) = i }) implies
    set_sInf(set_reindex_family(family, h)).subset(set_sInf(family))
} by {
    if forall(i: I) { h(g(i)) = i } {
        forall(i: I) {
            set_sInf_subset_family(set_reindex_family(family, h), g(i))
            set_sInf(set_reindex_family(family, h)).subset(set_reindex_family(family, h, g(i)))
            set_reindex_family(family, h, g(i)) = family(h(g(i)))
            h(g(i)) = i
            set_sInf(set_reindex_family(family, h)).subset(family(i))
        }
        set_subset_sInf_of_subset_family(set_sInf(set_reindex_family(family, h)), family)
        set_sInf(set_reindex_family(family, h)).subset(set_sInf(family))
    }
}

/// Reindexing by a map with a section preserves the family supremum.
theorem set_sSup_reindex_eq_of_section[K, I, J](family: I -> Set[K], h: J -> I, g: I -> J) {
    (forall(i: I) { h(g(i)) = i }) implies
    set_sSup(set_reindex_family(family, h)) = set_sSup(family)
} by {
    if forall(i: I) { h(g(i)) = i } {
        let u = set_sSup(set_reindex_family(family, h))
        let v = set_sSup(family)
        set_sSup_reindex_subset(family, h)
        u.subset(v)
        set_sSup_subset_reindex_of_section(family, h, g)
        set_sSup(family).subset(set_sSup(set_reindex_family(family, h)))
        v.subset(u)
        u.subset(v) and v.subset(u)
        subset_antisymm(u, v)
        set_sSup(set_reindex_family(family, h)) = set_sSup(family)
    }
}

/// Reindexing by a map with a section preserves the family infimum.
theorem set_sInf_reindex_eq_of_section[K, I, J](family: I -> Set[K], h: J -> I, g: I -> J) {
    (forall(i: I) { h(g(i)) = i }) implies
    set_sInf(set_reindex_family(family, h)) = set_sInf(family)
} by {
    if forall(i: I) { h(g(i)) = i } {
        let u = set_sInf(set_reindex_family(family, h))
        let v = set_sInf(family)
        set_sInf_subset_reindex(family, h)
        v.subset(u)
        set_sInf_reindex_subset_of_section(family, h, g)
        set_sInf(set_reindex_family(family, h)).subset(set_sInf(family))
        u.subset(v)
        u.subset(v) and v.subset(u)
        subset_antisymm(u, v)
        set_sInf(set_reindex_family(family, h)) = set_sInf(family)
    }
}

/// Reindexing by a map with a right inverse preserves the family supremum.
theorem set_sSup_reindex_eq_of_right_inverse[K, I, J](family: I -> Set[K],
    h: J -> I, g: I -> J) {
    is_right_inverse_fn(h, g) implies set_sSup(set_reindex_family(family, h)) = set_sSup(family)
} by {
    if is_right_inverse_fn(h, g) {
        is_right_inverse_fn(h, g) = forall(i: I) {
            h(g(i)) = i
        }
        set_sSup_reindex_eq_of_section(family, h, g)
        set_sSup(set_reindex_family(family, h)) = set_sSup(family)
    }
}

/// Reindexing by a map with a right inverse preserves the family infimum.
theorem set_sInf_reindex_eq_of_right_inverse[K, I, J](family: I -> Set[K],
    h: J -> I, g: I -> J) {
    is_right_inverse_fn(h, g) implies set_sInf(set_reindex_family(family, h)) = set_sInf(family)
} by {
    if is_right_inverse_fn(h, g) {
        is_right_inverse_fn(h, g) = forall(i: I) {
            h(g(i)) = i
        }
        set_sInf_reindex_eq_of_section(family, h, g)
        set_sInf(set_reindex_family(family, h)) = set_sInf(family)
    }
}

/// Reindexing by a surjective map preserves the family supremum.
theorem set_sSup_reindex_eq_of_surjective[K, I, J](family: I -> Set[K], h: J -> I) {
    is_surjective_fn(h) implies
    set_sSup(set_reindex_family(family, h)) = set_sSup(family)
} by {
    if is_surjective_fn(h) {
        let u = set_sSup(set_reindex_family(family, h))
        let v = set_sSup(family)
        set_sSup_reindex_subset(family, h)
        u.subset(v)
        forall(i: I) {
            surjective_fn_has_preimage(h, i)
            let j: J satisfy {
                h(j) = i
            }
            set_reindex_family(family, h, j) = family(h(j))
            set_reindex_family(family, h, j) = family(i)
            family(i) = set_reindex_family(family, h, j)
        }
        set_sSup_subset_of_family_occurs(family, set_reindex_family(family, h))
        set_sSup(family).subset(set_sSup(set_reindex_family(family, h)))
        v.subset(u)
        u.subset(v) and v.subset(u)
        subset_antisymm(u, v)
        set_sSup(set_reindex_family(family, h)) = set_sSup(family)
    }
}

/// Reindexing by a surjective map preserves the family infimum.
theorem set_sInf_reindex_eq_of_surjective[K, I, J](family: I -> Set[K], h: J -> I) {
    is_surjective_fn(h) implies
    set_sInf(set_reindex_family(family, h)) = set_sInf(family)
} by {
    if is_surjective_fn(h) {
        let u = set_sInf(set_reindex_family(family, h))
        let v = set_sInf(family)
        set_sInf_subset_reindex(family, h)
        v.subset(u)
        forall(i: I) {
            surjective_fn_has_preimage(h, i)
            let j: J satisfy {
                h(j) = i
            }
            set_reindex_family(family, h, j) = family(h(j))
            set_reindex_family(family, h, j) = family(i)
        }
        set_sInf_subset_of_family_occurs(set_reindex_family(family, h), family)
        set_sInf(set_reindex_family(family, h)).subset(set_sInf(family))
        u.subset(v)
        u.subset(v) and v.subset(u)
        subset_antisymm(u, v)
        set_sInf(set_reindex_family(family, h)) = set_sInf(family)
    }
}

/// Reindexing by a bijective map preserves the family supremum.
theorem set_sSup_reindex_eq_of_bijection[K, I, J](family: I -> Set[K], h: J -> I) {
    is_bijection_fn(h) implies
    set_sSup(set_reindex_family(family, h)) = set_sSup(family)
} by {
    if is_bijection_fn(h) {
        bijection_fn_is_surjective(h)
        is_surjective_fn(h)
        set_sSup_reindex_eq_of_surjective(family, h)
        set_sSup(set_reindex_family(family, h)) = set_sSup(family)
    }
}

/// Reindexing by a bijective map preserves the family infimum.
theorem set_sInf_reindex_eq_of_bijection[K, I, J](family: I -> Set[K], h: J -> I) {
    is_bijection_fn(h) implies
    set_sInf(set_reindex_family(family, h)) = set_sInf(family)
} by {
    if is_bijection_fn(h) {
        bijection_fn_is_surjective(h)
        is_surjective_fn(h)
        set_sInf_reindex_eq_of_surjective(family, h)
        set_sInf(set_reindex_family(family, h)) = set_sInf(family)
    }
}

/// Preimage commutes with indexed set suprema.
theorem set_preimage_sSup[T, U, I](f: T -> U, family: I -> Set[U]) {
    set_preimage(f, set_sSup(family)) = set_sSup(set_preimage_family(f, family))
} by {
    let u = set_preimage(f, set_sSup(family))
    let v = set_sSup(set_preimage_family(f, family))
    forall(x: T) {
        if u.contains(x) {
            set_preimage_contains_eq(f, set_sSup(family), x)
            set_sSup(family).contains(f(x))
            set_sSup_contains_eq(family, f(x))
            let i: I satisfy {
                family(i).contains(f(x))
            }
            set_preimage_contains_eq(f, family(i), x)
            set_preimage(f, family(i)).contains(x)
            set_preimage_family(f, family, i) = set_preimage(f, family(i))
            set_preimage_family(f, family, i).contains(x)
            set_sSup_contains_eq(set_preimage_family(f, family), x)
            v.contains(x)
        }
        if v.contains(x) {
            set_sSup_contains_eq(set_preimage_family(f, family), x)
            let i: I satisfy {
                set_preimage_family(f, family, i).contains(x)
            }
            set_preimage_family(f, family, i) = set_preimage(f, family(i))
            set_preimage_contains_eq(f, family(i), x)
            family(i).contains(f(x))
            set_sSup_contains_eq(family, f(x))
            set_sSup(family).contains(f(x))
            set_preimage_contains_eq(f, set_sSup(family), x)
            u.contains(x)
        }
        u.contains(x) = v.contains(x)
    }
    set_ext(u, v)
}

/// Preimage commutes with indexed set infima.
theorem set_preimage_sInf[T, U, I](f: T -> U, family: I -> Set[U]) {
    set_preimage(f, set_sInf(family)) = set_sInf(set_preimage_family(f, family))
} by {
    let u = set_preimage(f, set_sInf(family))
    let v = set_sInf(set_preimage_family(f, family))
    forall(x: T) {
        if u.contains(x) {
            set_preimage_contains_eq(f, set_sInf(family), x)
            set_sInf(family).contains(f(x))
            set_sInf_contains_eq(family, f(x))
            forall(i: I) {
                family(i).contains(f(x))
                set_preimage_contains_eq(f, family(i), x)
                set_preimage(f, family(i)).contains(x)
                set_preimage_family(f, family, i).contains(x)
            }
            set_sInf_contains_eq(set_preimage_family(f, family), x)
            v.contains(x)
        }
        if v.contains(x) {
            set_sInf_contains_eq(set_preimage_family(f, family), x)
            forall(i: I) {
                set_preimage_family(f, family, i).contains(x)
                set_preimage_family(f, family, i) = set_preimage(f, family(i))
                set_preimage_contains_eq(f, family(i), x)
                family(i).contains(f(x))
            }
            set_sInf_contains_eq(family, f(x))
            set_sInf(family).contains(f(x))
            set_preimage_contains_eq(f, set_sInf(family), x)
            u.contains(x)
        }
        u.contains(x) = v.contains(x)
    }
    set_ext(u, v)
}

/// A preimage of a family supremum is contained in a set exactly when every member preimage is contained in it.
theorem set_preimage_sSup_subset_iff[T, U, I](f: T -> U, family: I -> Set[U], s: Set[T]) {
    set_preimage(f, set_sSup(family)).subset(s) = forall(i: I) {
        set_preimage_family(f, family, i).subset(s)
    }
} by {
    set_preimage_sSup(f, family)
    set_preimage(f, set_sSup(family)) = set_sSup(set_preimage_family(f, family))
    set_sSup_subset_iff(set_preimage_family(f, family), s)
    set_preimage(f, set_sSup(family)).subset(s) = forall(i: I) {
        set_preimage_family(f, family, i).subset(s)
    }
}

/// A set is contained in a preimage of a family infimum exactly when it is contained in every member preimage.
theorem set_subset_preimage_sInf_iff[T, U, I](s: Set[T], f: T -> U, family: I -> Set[U]) {
    s.subset(set_preimage(f, set_sInf(family))) = forall(i: I) {
        s.subset(set_preimage_family(f, family, i))
    }
} by {
    set_preimage_sInf(f, family)
    set_preimage(f, set_sInf(family)) = set_sInf(set_preimage_family(f, family))
    set_subset_sInf_iff(s, set_preimage_family(f, family))
    s.subset(set_preimage(f, set_sInf(family))) = forall(i: I) {
        s.subset(set_preimage_family(f, family, i))
    }
}

/// Image commutes with indexed set suprema.
theorem set_image_sSup[T, U, I](family: I -> Set[T], f: T -> U) {
    set_image(set_sSup(family), f) = set_sSup(set_image_family(family, f))
} by {
    let u = set_image(set_sSup(family), f)
    let v = set_sSup(set_image_family(family, f))
    forall(y: U) {
        if u.contains(y) {
            set_image_contains_witness(set_sSup(family), f, y)
            let x: T satisfy {
                set_sSup(family).contains(x) and y = f(x)
            }
            set_sSup_contains_eq(family, x)
            let i: I satisfy {
                family(i).contains(x)
            }
            maps_into_set_image(family(i), f, x)
            set_image(family(i), f).contains(f(x))
            set_image_family(family, f, i) = set_image(family(i), f)
            set_image_family(family, f, i).contains(y)
            set_sSup_contains_eq(set_image_family(family, f), y)
            v.contains(y)
        }
        if v.contains(y) {
            set_sSup_contains_eq(set_image_family(family, f), y)
            let i: I satisfy {
                set_image_family(family, f, i).contains(y)
            }
            set_image_family(family, f, i) = set_image(family(i), f)
            set_image_contains_witness(family(i), f, y)
            let x: T satisfy {
                family(i).contains(x) and y = f(x)
            }
            set_sSup_contains_eq(family, x)
            set_sSup(family).contains(x)
            maps_into_set_image(set_sSup(family), f, x)
            u.contains(f(x))
            u.contains(y)
        }
        u.contains(y) = v.contains(y)
    }
    set_ext(u, v)
}

/// Image of a family supremum is contained in a target exactly when every member maps into it.
theorem set_image_sSup_subset_iff[T, U, I](family: I -> Set[T], s: Set[U], f: T -> U) {
    set_image(set_sSup(family), f).subset(s) = forall(i: I) {
        family(i).subset(set_preimage(f, s))
    }
} by {
    set_image_subset_iff_subset_preimage(set_sSup(family), s, f)
    set_image(set_sSup(family), f).subset(s) = set_sSup(family).subset(set_preimage(f, s))
    set_sSup_subset_iff(family, set_preimage(f, s))
    set_image(set_sSup(family), f).subset(s) = forall(i: I) {
        family(i).subset(set_preimage(f, s))
    }
}

/// Image of a family infimum is contained in the infimum of the images.
theorem set_image_sInf_subset[T, U, I](family: I -> Set[T], f: T -> U) {
    set_image(set_sInf(family), f).subset(set_sInf(set_image_family(family, f)))
} by {
    forall(i: I) {
        set_sInf_subset_family(family, i)
        set_image_monotone(set_sInf(family), family(i), f)
        set_image(set_sInf(family), f).subset(set_image(family(i), f))
        set_image_family(family, f, i) = set_image(family(i), f)
        set_image(set_sInf(family), f).subset(set_image_family(family, f, i))
    }
    set_subset_sInf_of_subset_family(set_image(set_sInf(family), f), set_image_family(family, f))
    set_image(set_sInf(family), f).subset(set_sInf(set_image_family(family, f)))
}

/// Image of a flattened binary-family infimum is contained in the infimum of the member images.
theorem set_image_sInf_pair_family_subset[T, U, I, J](family: (I, J) -> Set[T],
    f: T -> U) {
    set_image(set_sInf(set_pair_family(family)), f).subset(
        set_sInf(set_pair_family(set_image_binary_family(family, f))))
} by {
    let a = set_image_family(set_pair_family(family), f)
    let b = set_pair_family(set_image_binary_family(family, f))
    set_image_sInf_subset(set_pair_family(family), f)
    set_image(set_sInf(set_pair_family(family)), f).subset(set_sInf(a))
    forall(p: Pair[I, J]) {
        a(p) = set_image(set_pair_family(family, p), f)
        set_pair_family(family, p) = family(p.first, p.second)
        set_image(family(p.first, p.second), f) =
            set_image_binary_family(family, f, p.first, p.second)
        b(p) = set_image_binary_family(family, f, p.first, p.second)
        a(p) = b(p)
    }
    set_sInf_eq_of_family_subset(a, b)
    set_sInf(a) = set_sInf(b)
    subset_trans(set_image(set_sInf(set_pair_family(family)), f), set_sInf(a), set_sInf(b))
    set_image(set_sInf(set_pair_family(family)), f).subset(
        set_sInf(set_pair_family(set_image_binary_family(family, f))))
}

/// Under an injective map, the infimum of an inhabited binary family of images is contained in the image of the infimum.
theorem set_image_sInf_pair_family_reverse_of_injective[T, U, I, J](
    family: (I, J) -> Set[T], f: T -> U, i0: I, j0: J
) {
    is_injective_fn(f) implies
    set_sInf(set_pair_family(set_image_binary_family(family, f))).subset(
        set_image(set_sInf(set_pair_family(family)), f))
} by {
    if is_injective_fn(f) {
        forall(y: U) {
            if set_sInf(set_pair_family(set_image_binary_family(family, f))).contains(y) {
                set_sInf_pair_family_contains_eq(set_image_binary_family(family, f), y)
                set_image_binary_family(family, f, i0, j0).contains(y)
                set_image_binary_family(family, f, i0, j0) = set_image(family(i0, j0), f)
                set_image_contains_witness(family(i0, j0), f, y)
                let x: T satisfy {
                    family(i0, j0).contains(x) and y = f(x)
                }
                forall(i: I, j: J) {
                    set_image_binary_family(family, f, i, j).contains(y)
                    set_image_binary_family(family, f, i, j) = set_image(family(i, j), f)
                    set_image_contains_witness(family(i, j), f, y)
                    let xi: T satisfy {
                        family(i, j).contains(xi) and y = f(xi)
                    }
                    f(x) = f(xi)
                    injective_fn_eq(f, x, xi)
                    x = xi
                    family(i, j).contains(x)
                }
                set_sInf_pair_family_contains_eq(family, x)
                set_sInf(set_pair_family(family)).contains(x)
                maps_into_set_image(set_sInf(set_pair_family(family)), f, x)
                set_image(set_sInf(set_pair_family(family)), f).contains(f(x))
                set_image(set_sInf(set_pair_family(family)), f).contains(y)
            }
        }
    }
}

/// An injective map preserves infima of inhabited binary set families by image.
theorem set_image_sInf_pair_family_of_injective[T, U, I, J](
    family: (I, J) -> Set[T], f: T -> U, i0: I, j0: J
) {
    is_injective_fn(f) implies
    set_image(set_sInf(set_pair_family(family)), f) =
        set_sInf(set_pair_family(set_image_binary_family(family, f)))
} by {
    if is_injective_fn(f) {
        let u = set_image(set_sInf(set_pair_family(family)), f)
        let v = set_sInf(set_pair_family(set_image_binary_family(family, f)))
        set_image_sInf_pair_family_subset(family, f)
        u.subset(v)
        set_image_sInf_pair_family_reverse_of_injective(family, f, i0, j0)
        v.subset(u)
        u.subset(v) and v.subset(u)
        subset_antisymm(u, v)
        set_image(set_sInf(set_pair_family(family)), f) =
            set_sInf(set_pair_family(set_image_binary_family(family, f)))
    }
}

/// A set is contained in an injective image of a binary-family infimum exactly when it is contained in every member image.
theorem set_subset_image_sInf_pair_family_iff_of_injective[T, U, I, J](
    s: Set[U], family: (I, J) -> Set[T], f: T -> U, i0: I, j0: J
) {
    is_injective_fn(f) implies
    s.subset(set_image(set_sInf(set_pair_family(family)), f)) = forall(i: I) {
        forall(j: J) {
            s.subset(set_image_binary_family(family, f, i, j))
        }
    }
} by {
    if is_injective_fn(f) {
        set_image_sInf_pair_family_of_injective(family, f, i0, j0)
        set_image(set_sInf(set_pair_family(family)), f) =
            set_sInf(set_pair_family(set_image_binary_family(family, f)))
        set_subset_sInf_pair_family_iff(s, set_image_binary_family(family, f))
        s.subset(set_image(set_sInf(set_pair_family(family)), f)) = forall(i: I) {
            forall(j: J) {
                s.subset(set_image_binary_family(family, f, i, j))
            }
        }
    }
}

/// Under an injective map, the infimum of an inhabited family of images is contained in the image of the infimum.
theorem set_image_sInf_reverse_of_injective[T, U, I: Inhabited](family: I -> Set[T],
    f: T -> U) {
    is_injective_fn(f) implies
    set_sInf(set_image_family(family, f)).subset(set_image(set_sInf(family), f))
} by {
    if is_injective_fn(f) {
        let i0: I satisfy {
            true
        }
        forall(y: U) {
            if set_sInf(set_image_family(family, f)).contains(y) {
                set_sInf_contains_eq(set_image_family(family, f), y)
                set_image_family(family, f, i0).contains(y)
                set_image_family(family, f, i0) = set_image(family(i0), f)
                set_image_contains_witness(family(i0), f, y)
                let x: T satisfy {
                    family(i0).contains(x) and y = f(x)
                }
                forall(i: I) {
                    set_image_family(family, f, i).contains(y)
                    set_image_family(family, f, i) = set_image(family(i), f)
                    set_image_contains_witness(family(i), f, y)
                    let xi: T satisfy {
                        family(i).contains(xi) and y = f(xi)
                    }
                    f(x) = f(xi)
                    injective_fn_eq(f, x, xi)
                    x = xi
                    family(i).contains(x)
                }
                set_sInf_contains_eq(family, x)
                set_sInf(family).contains(x)
                maps_into_set_image(set_sInf(family), f, x)
                set_image(set_sInf(family), f).contains(f(x))
                set_image(set_sInf(family), f).contains(y)
            }
        }
    }
}

/// An injective map preserves infima of inhabited set families by image.
theorem set_image_sInf_of_injective[T, U, I: Inhabited](family: I -> Set[T], f: T -> U) {
    is_injective_fn(f) implies
    set_image(set_sInf(family), f) = set_sInf(set_image_family(family, f))
} by {
    if is_injective_fn(f) {
        set_image_sInf_subset(family, f)
        set_image_sInf_reverse_of_injective(family, f)
        subset_antisymm(set_image(set_sInf(family), f), set_sInf(set_image_family(family, f)))
    }
}

/// Under a bijective map, the infimum of a family of images is contained in the image of the infimum.
theorem set_image_sInf_reverse_of_bijection[T: Inhabited, U, I](family: I -> Set[T],
    f: T -> U) {
    is_bijection_fn(f) implies
    set_sInf(set_image_family(family, f)).subset(set_image(set_sInf(family), f))
} by {
    if is_bijection_fn(f) {
        bijection_fn_is_surjective(f)
        is_surjective_fn(f)
        forall(y: U) {
            if set_sInf(set_image_family(family, f)).contains(y) {
                set_sInf_contains_eq(set_image_family(family, f), y)
                forall(i: I) {
                    set_image_family(family, f, i).contains(y)
                    set_image_family(family, f, i) = set_image(family(i), f)
                    set_image_contains_inverse_of_bijection(family(i), f, y)
                    family(i).contains(inverse_fn(f, y))
                }
                set_sInf_contains_eq(family, inverse_fn(f, y))
                set_sInf(family).contains(inverse_fn(f, y))
                set_inverse_contains_imp_image_contains(set_sInf(family), f, y)
                set_image(set_sInf(family), f).contains(y)
            }
        }
    }
}

/// A bijective map preserves arbitrary set-family infima by image.
theorem set_image_sInf_of_bijection[T: Inhabited, U, I](family: I -> Set[T], f: T -> U) {
    is_bijection_fn(f) implies
    set_image(set_sInf(family), f) = set_sInf(set_image_family(family, f))
} by {
    if is_bijection_fn(f) {
        let u = set_image(set_sInf(family), f)
        let v = set_sInf(set_image_family(family, f))
        set_image_sInf_subset(family, f)
        u.subset(v)
        set_image_sInf_reverse_of_bijection(family, f)
        v.subset(u)
        u.subset(v) and v.subset(u)
        subset_antisymm(u, v)
        set_image(set_sInf(family), f) = set_sInf(set_image_family(family, f))
    }
}

/// A set is contained in an injective image of an inhabited family infimum exactly when it is contained in every member image.
theorem set_subset_image_sInf_iff_of_injective[T, U, I: Inhabited](s: Set[U],
    family: I -> Set[T], f: T -> U) {
    is_injective_fn(f) implies
    s.subset(set_image(set_sInf(family), f)) = forall(i: I) {
        s.subset(set_image_family(family, f, i))
    }
} by {
    if is_injective_fn(f) {
        set_image_sInf_of_injective(family, f)
        set_image(set_sInf(family), f) = set_sInf(set_image_family(family, f))
        set_subset_sInf_iff(s, set_image_family(family, f))
        s.subset(set_image(set_sInf(family), f)) = forall(i: I) {
            s.subset(set_image_family(family, f, i))
        }
    }
}

/// A set is contained in a bijective image of a family infimum exactly when it is contained in every member image.
theorem set_subset_image_sInf_iff_of_bijection[T: Inhabited, U, I](s: Set[U],
    family: I -> Set[T], f: T -> U) {
    is_bijection_fn(f) implies
    s.subset(set_image(set_sInf(family), f)) = forall(i: I) {
        s.subset(set_image_family(family, f, i))
    }
} by {
    if is_bijection_fn(f) {
        set_image_sInf_of_bijection(family, f)
        set_image(set_sInf(family), f) = set_sInf(set_image_family(family, f))
        set_subset_sInf_iff(s, set_image_family(family, f))
        s.subset(set_image(set_sInf(family), f)) = forall(i: I) {
            s.subset(set_image_family(family, f, i))
        }
    }
}

/// A bijective map preserves infima of binary set families by image.
theorem set_image_sInf_pair_family_of_bijection[T: Inhabited, U, I, J](
    family: (I, J) -> Set[T], f: T -> U
) {
    is_bijection_fn(f) implies
    set_image(set_sInf(set_pair_family(family)), f) =
        set_sInf(set_pair_family(set_image_binary_family(family, f)))
} by {
    if is_bijection_fn(f) {
        let a = set_image_family(set_pair_family(family), f)
        let b = set_pair_family(set_image_binary_family(family, f))
        set_image_sInf_of_bijection(set_pair_family(family), f)
        set_image(set_sInf(set_pair_family(family)), f) = set_sInf(a)
        forall(p: Pair[I, J]) {
            a(p) = set_image(set_pair_family(family, p), f)
            set_pair_family(family, p) = family(p.first, p.second)
            set_image(family(p.first, p.second), f) =
                set_image_binary_family(family, f, p.first, p.second)
            b(p) = set_image_binary_family(family, f, p.first, p.second)
            a(p) = b(p)
        }
        set_sInf_eq_of_family_subset(a, b)
        set_sInf(a) = set_sInf(b)
        set_image(set_sInf(set_pair_family(family)), f) =
            set_sInf(set_pair_family(set_image_binary_family(family, f)))
    }
}

/// A set is contained in a bijective image of a binary-family infimum exactly when it is contained in every member image.
theorem set_subset_image_sInf_pair_family_iff_of_bijection[T: Inhabited, U, I, J](
    s: Set[U], family: (I, J) -> Set[T], f: T -> U
) {
    is_bijection_fn(f) implies
    s.subset(set_image(set_sInf(set_pair_family(family)), f)) = forall(i: I) {
        forall(j: J) {
            s.subset(set_image_binary_family(family, f, i, j))
        }
    }
} by {
    if is_bijection_fn(f) {
        set_image_sInf_pair_family_of_bijection(family, f)
        set_image(set_sInf(set_pair_family(family)), f) =
            set_sInf(set_pair_family(set_image_binary_family(family, f)))
        set_subset_sInf_pair_family_iff(s, set_image_binary_family(family, f))
        s.subset(set_image(set_sInf(set_pair_family(family)), f)) = forall(i: I) {
            forall(j: J) {
                s.subset(set_image_binary_family(family, f, i, j))
            }
        }
    }
}

/// A family supremum is the least upper bound of the family.
theorem set_sSup_is_lub[K, I](family: I -> Set[K], s: Set[K]) {
    (forall(i: I) { family(i).subset(set_sSup(family)) }) and
    (set_sSup(family).subset(s) = forall(i: I) { family(i).subset(s) })
} by {
    forall(i: I) {
        set_family_subset_sSup(family, i)
    }
    set_sSup_subset_iff(family, s)
}

/// A family infimum is the greatest lower bound of the family.
theorem set_sInf_is_glb[K, I](family: I -> Set[K], s: Set[K]) {
    (forall(i: I) { set_sInf(family).subset(family(i)) }) and
    (s.subset(set_sInf(family)) = forall(i: I) { s.subset(family(i)) })
} by {
    forall(i: I) {
        set_sInf_subset_family(family, i)
    }
    set_subset_sInf_iff(s, family)
}

/// The supremum of an empty family is empty when every member is empty.
theorem set_sSup_empty_of_forall_empty[K, I](family: I -> Set[K]) {
    (forall(i: I) { family(i) = Set[K].empty_set }) implies
    set_sSup(family) = Set[K].empty_set
} by {
    if forall(i: I) { family(i) = Set[K].empty_set } {
        indexed_union_empty_of_forall_empty(family)
        indexed_union(family) = Set[K].empty_set
        set_sSup(family) = Set[K].empty_set
    }
}

/// If a family supremum is empty, every member is empty.
theorem set_sSup_empty_imp_forall_empty[K, I](family: I -> Set[K]) {
    set_sSup(family) = Set[K].empty_set implies forall(i: I) {
        family(i) = Set[K].empty_set
    }
} by {
    if set_sSup(family) = Set[K].empty_set {
        set_sSup(family) = indexed_union(family)
        indexed_union(family) = Set[K].empty_set
        indexed_union_empty_imp_forall_empty(family)
    }
}

/// A family supremum is empty exactly when every member is empty.
theorem set_sSup_empty_iff_forall_empty[K, I](family: I -> Set[K]) {
    set_sSup(family) = Set[K].empty_set = forall(i: I) {
        family(i) = Set[K].empty_set
    }
} by {
    if set_sSup(family) = Set[K].empty_set {
        set_sSup_empty_imp_forall_empty(family)
        forall(i: I) {
            family(i) = Set[K].empty_set
        }
    }
    if forall(i: I) { family(i) = Set[K].empty_set } {
        set_sSup_empty_of_forall_empty(family)
        set_sSup(family) = Set[K].empty_set
    }
    set_sSup(family) = Set[K].empty_set = forall(i: I) {
        family(i) = Set[K].empty_set
    }
}

/// The supremum of the constantly empty family is empty.
theorem set_sSup_empty_family[K, I] {
    set_sSup(set_empty_family[K, I]) = Set[K].empty_set
} by {
    set_sSup_empty_of_forall_empty(set_empty_family[K, I])
}

/// The infimum of universal sets is universal.
theorem set_sInf_universal_of_forall_universal[K, I](family: I -> Set[K]) {
    (forall(i: I) { family(i) = Set[K].universal_set }) implies
    set_sInf(family) = Set[K].universal_set
} by {
    if forall(i: I) { family(i) = Set[K].universal_set } {
        indexed_intersection_universal_of_forall_universal(family)
        indexed_intersection(family) = Set[K].universal_set
        set_sInf(family) = Set[K].universal_set
    }
}

/// If a family infimum is universal, every member is universal.
theorem set_sInf_universal_imp_forall_universal[K, I](family: I -> Set[K]) {
    set_sInf(family) = Set[K].universal_set implies forall(i: I) {
        family(i) = Set[K].universal_set
    }
} by {
    if set_sInf(family) = Set[K].universal_set {
        set_sInf(family) = indexed_intersection(family)
        indexed_intersection(family) = Set[K].universal_set
        indexed_intersection_universal_imp_forall_universal(family)
    }
}

/// A family infimum is universal exactly when every member is universal.
theorem set_sInf_universal_iff_forall_universal[K, I](family: I -> Set[K]) {
    set_sInf(family) = Set[K].universal_set = forall(i: I) {
        family(i) = Set[K].universal_set
    }
} by {
    if set_sInf(family) = Set[K].universal_set {
        set_sInf_universal_imp_forall_universal(family)
        forall(i: I) {
            family(i) = Set[K].universal_set
        }
    }
    if forall(i: I) { family(i) = Set[K].universal_set } {
        set_sInf_universal_of_forall_universal(family)
        set_sInf(family) = Set[K].universal_set
    }
    set_sInf(family) = Set[K].universal_set = forall(i: I) {
        family(i) = Set[K].universal_set
    }
}

/// The infimum of the constantly universal family is universal.
theorem set_sInf_universal_family[K, I] {
    set_sInf(set_universal_family[K, I]) = Set[K].universal_set
} by {
    set_sInf_universal_of_forall_universal(set_universal_family[K, I])
}

/// The supremum of a constant family is contained in the constant member.
theorem set_sSup_constant_family_subset[K, I](s: Set[K]) {
    set_sSup(set_constant_family[K, I](s)).subset(s)
} by {
    forall(i: I) {
        set_constant_family(s, i) = s
        set_constant_family(s, i).subset(s)
    }
    set_sSup_subset_of_family_subset(set_constant_family[K, I](s), s)
}

/// In an inhabited family, the constant member is contained in its supremum.
theorem set_subset_sSup_constant_family[K, I: Inhabited](s: Set[K]) {
    s.subset(set_sSup(set_constant_family[K, I](s)))
} by {
    let i: I satisfy {
        true
    }
    set_family_subset_sSup(set_constant_family[K, I](s), i)
    set_constant_family(s, i) = s
    s.subset(set_sSup(set_constant_family[K, I](s)))
}

/// In an inhabited family, the supremum of a constant family is the constant member.
theorem set_sSup_constant_family[K, I: Inhabited](s: Set[K]) {
    set_sSup(set_constant_family[K, I](s)) = s
} by {
    set_sSup_constant_family_subset[K, I](s)
    set_subset_sSup_constant_family[K, I](s)
    subset_antisymm(set_sSup(set_constant_family[K, I](s)), s)
}

/// The constant member is contained in the infimum of a constant family.
theorem set_subset_sInf_constant_family[K, I](s: Set[K]) {
    s.subset(set_sInf(set_constant_family[K, I](s)))
} by {
    forall(i: I) {
        set_constant_family(s, i) = s
        s.subset(set_constant_family(s, i))
    }
    set_subset_sInf_of_subset_family(s, set_constant_family[K, I](s))
}

/// In an inhabited family, the infimum of a constant family is contained in the constant member.
theorem set_sInf_constant_family_subset[K, I: Inhabited](s: Set[K]) {
    set_sInf(set_constant_family[K, I](s)).subset(s)
} by {
    let i: I satisfy {
        true
    }
    set_sInf_subset_family(set_constant_family[K, I](s), i)
    set_constant_family(s, i) = s
    set_sInf(set_constant_family[K, I](s)).subset(s)
}

/// In an inhabited family, the infimum of a constant family is the constant member.
theorem set_sInf_constant_family[K, I: Inhabited](s: Set[K]) {
    set_sInf(set_constant_family[K, I](s)) = s
} by {
    set_sInf_constant_family_subset[K, I](s)
    set_subset_sInf_constant_family[K, I](s)
    subset_antisymm(set_sInf(set_constant_family[K, I](s)), s)
}

/// The supremum of complements is the complement of the infimum.
theorem set_sSup_complement_eq_sInf_complement[K, I](family: I -> Set[K]) {
    set_sSup(function(i: I) { family(i).c }) = set_sInf(family).c
} by {
    indexed_union_complement_eq_intersection_complement(family)
    indexed_union(function(i: I) { family(i).c }) = indexed_intersection(family).c
    set_sSup(function(i: I) { family(i).c }) = set_sInf(family).c
}

/// The infimum of complements is the complement of the supremum.
theorem set_sInf_complement_eq_sSup_complement[K, I](family: I -> Set[K]) {
    set_sInf(function(i: I) { family(i).c }) = set_sSup(family).c
} by {
    indexed_intersection_complement_eq_union_complement(family)
    indexed_intersection(function(i: I) { family(i).c }) = indexed_union(family).c
    set_sInf(function(i: I) { family(i).c }) = set_sSup(family).c
}

/// The family formed by taking the infimum with a fixed set.
define set_inf_family[K, I](s: Set[K], family: I -> Set[K], i: I) -> Set[K] {
    set_inf(s, family(i))
}

/// The family formed by taking the supremum with a fixed set.
define set_sup_family[K, I](s: Set[K], family: I -> Set[K], i: I) -> Set[K] {
    set_sup(s, family(i))
}

/// The family formed by taking the infimum with a fixed set on the right.
define set_inf_family_right[K, I](family: I -> Set[K], s: Set[K], i: I) -> Set[K] {
    set_inf(family(i), s)
}

/// The family formed by taking the supremum with a fixed set on the right.
define set_sup_family_right[K, I](family: I -> Set[K], s: Set[K], i: I) -> Set[K] {
    set_sup(family(i), s)
}

/// Binary infimum distributes over a family supremum on the right.
theorem set_inf_sSup_distrib_left[K, I](s: Set[K], family: I -> Set[K]) {
    set_inf(s, set_sSup(family)) = set_sSup(set_inf_family(s, family))
} by {
    let u = set_inf(s, set_sSup(family))
    let v = set_sSup(set_inf_family(s, family))
    forall(x: K) {
        if u.contains(x) {
            set_inf_contains_eq(s, set_sSup(family), x)
            s.contains(x)
            set_sSup(family).contains(x)
            set_sSup_contains_eq(family, x)
            let i: I satisfy {
                family(i).contains(x)
            }
            set_inf_contains_eq(s, family(i), x)
            set_inf(s, family(i)).contains(x)
            set_inf_family(s, family, i) = set_inf(s, family(i))
            set_inf_family(s, family, i).contains(x)
            exists(j: I) {
                j = i and set_inf_family(s, family, j).contains(x)
            }
            set_sSup_contains_eq(set_inf_family(s, family), x)
            v.contains(x)
        }
        if v.contains(x) {
            set_sSup_contains_eq(set_inf_family(s, family), x)
            let i: I satisfy {
                set_inf_family(s, family, i).contains(x)
            }
            set_inf_family(s, family, i) = set_inf(s, family(i))
            set_inf_contains_eq(s, family(i), x)
            s.contains(x)
            family(i).contains(x)
            set_sSup_contains_eq(family, x)
            set_sSup(family).contains(x)
            set_inf_contains_eq(s, set_sSup(family), x)
            u.contains(x)
        }
        u.contains(x) = v.contains(x)
    }
    set_ext(u, v)
}

/// Binary infimum distributes over a family supremum on the left.
theorem set_inf_sSup_distrib_right[K, I](family: I -> Set[K], s: Set[K]) {
    set_inf(set_sSup(family), s) = set_sSup(set_inf_family_right(family, s))
} by {
    let u = set_inf(set_sSup(family), s)
    let v = set_sSup(set_inf_family_right(family, s))
    forall(x: K) {
        if u.contains(x) {
            set_inf_contains_eq(set_sSup(family), s, x)
            set_sSup(family).contains(x)
            s.contains(x)
            set_sSup_contains_eq(family, x)
            let i: I satisfy {
                family(i).contains(x)
            }
            set_inf_contains_eq(family(i), s, x)
            set_inf(family(i), s).contains(x)
            set_inf_family_right(family, s, i) = set_inf(family(i), s)
            set_inf_family_right(family, s, i).contains(x)
            exists(j: I) {
                j = i and set_inf_family_right(family, s, j).contains(x)
            }
            set_sSup_contains_eq(set_inf_family_right(family, s), x)
            v.contains(x)
        }
        if v.contains(x) {
            set_sSup_contains_eq(set_inf_family_right(family, s), x)
            let i: I satisfy {
                set_inf_family_right(family, s, i).contains(x)
            }
            set_inf_family_right(family, s, i) = set_inf(family(i), s)
            set_inf_contains_eq(family(i), s, x)
            family(i).contains(x)
            s.contains(x)
            set_sSup_contains_eq(family, x)
            set_sSup(family).contains(x)
            set_inf_contains_eq(set_sSup(family), s, x)
            u.contains(x)
        }
        u.contains(x) = v.contains(x)
    }
    set_ext(u, v)
}

/// Binary supremum distributes over a family infimum on the right.
theorem set_sup_sInf_distrib_left[K, I](s: Set[K], family: I -> Set[K]) {
    set_sup(s, set_sInf(family)) = set_sInf(set_sup_family(s, family))
} by {
    let u = set_sup(s, set_sInf(family))
    let v = set_sInf(set_sup_family(s, family))
    forall(x: K) {
        if u.contains(x) {
            set_sup_contains_eq(s, set_sInf(family), x)
            if s.contains(x) {
                forall(i: I) {
                    set_sup_contains_eq(s, family(i), x)
                    set_sup(s, family(i)).contains(x)
                    set_sup_family(s, family, i).contains(x)
                }
                set_sInf_contains_eq(set_sup_family(s, family), x)
                v.contains(x)
            } else {
                set_sInf(family).contains(x)
                set_sInf_contains_eq(family, x)
                forall(i: I) {
                    family(i).contains(x)
                    set_sup_contains_eq(s, family(i), x)
                    set_sup(s, family(i)).contains(x)
                    set_sup_family(s, family, i).contains(x)
                }
                set_sInf_contains_eq(set_sup_family(s, family), x)
                v.contains(x)
            }
        }
        if v.contains(x) {
            set_sInf_contains_eq(set_sup_family(s, family), x)
            if s.contains(x) {
                set_sup_contains_eq(s, set_sInf(family), x)
                u.contains(x)
            } else {
                forall(i: I) {
                    set_sup_family(s, family, i).contains(x)
                    set_sup_family(s, family, i) = set_sup(s, family(i))
                    set_sup_contains_eq(s, family(i), x)
                    family(i).contains(x)
                }
                set_sInf_contains_eq(family, x)
                set_sInf(family).contains(x)
                set_sup_contains_eq(s, set_sInf(family), x)
                u.contains(x)
            }
        }
        u.contains(x) = v.contains(x)
    }
    set_ext(u, v)
}

/// Binary supremum distributes over a family infimum on the left.
theorem set_sup_sInf_distrib_right[K, I](family: I -> Set[K], s: Set[K]) {
    set_sup(set_sInf(family), s) = set_sInf(set_sup_family_right(family, s))
} by {
    let u = set_sup(set_sInf(family), s)
    let v = set_sInf(set_sup_family_right(family, s))
    forall(x: K) {
        if u.contains(x) {
            set_sup_contains_eq(set_sInf(family), s, x)
            if s.contains(x) {
                forall(i: I) {
                    set_sup_contains_eq(family(i), s, x)
                    set_sup(family(i), s).contains(x)
                    set_sup_family_right(family, s, i).contains(x)
                }
                set_sInf_contains_eq(set_sup_family_right(family, s), x)
                v.contains(x)
            } else {
                set_sInf(family).contains(x)
                set_sInf_contains_eq(family, x)
                forall(i: I) {
                    family(i).contains(x)
                    set_sup_contains_eq(family(i), s, x)
                    set_sup(family(i), s).contains(x)
                    set_sup_family_right(family, s, i).contains(x)
                }
                set_sInf_contains_eq(set_sup_family_right(family, s), x)
                v.contains(x)
            }
        }
        if v.contains(x) {
            set_sInf_contains_eq(set_sup_family_right(family, s), x)
            if s.contains(x) {
                set_sup_contains_eq(set_sInf(family), s, x)
                u.contains(x)
            } else {
                forall(i: I) {
                    set_sup_family_right(family, s, i).contains(x)
                    set_sup_family_right(family, s, i) = set_sup(family(i), s)
                    set_sup_contains_eq(family(i), s, x)
                    family(i).contains(x)
                }
                set_sInf_contains_eq(family, x)
                set_sInf(family).contains(x)
                set_sup_contains_eq(set_sInf(family), s, x)
                u.contains(x)
            }
        }
        u.contains(x) = v.contains(x)
    }
    set_ext(u, v)
}

/// Binary supremum distributes over a family supremum on the right.
theorem set_sup_sSup_distrib_left[K, I: Inhabited](s: Set[K], family: I -> Set[K]) {
    set_sup(s, set_sSup(family)) = set_sSup(set_sup_family(s, family))
} by {
    let u = set_sup(s, set_sSup(family))
    let v = set_sSup(set_sup_family(s, family))
    forall(x: K) {
        if u.contains(x) {
            set_sup_contains_eq(s, set_sSup(family), x)
            if s.contains(x) {
                let i: I satisfy {
                    true
                }
                set_sup_contains_eq(s, family(i), x)
                set_sup(s, family(i)).contains(x)
                set_sup_family(s, family, i) = set_sup(s, family(i))
                set_sup_family(s, family, i).contains(x)
                set_sSup_contains_eq(set_sup_family(s, family), x)
                v.contains(x)
            } else {
                set_sSup(family).contains(x)
                set_sSup_contains_eq(family, x)
                let i: I satisfy {
                    family(i).contains(x)
                }
                set_sup_contains_eq(s, family(i), x)
                set_sup(s, family(i)).contains(x)
                set_sup_family(s, family, i) = set_sup(s, family(i))
                set_sup_family(s, family, i).contains(x)
                set_sSup_contains_eq(set_sup_family(s, family), x)
                v.contains(x)
            }
        }
        if v.contains(x) {
            set_sSup_contains_eq(set_sup_family(s, family), x)
            let i: I satisfy {
                set_sup_family(s, family, i).contains(x)
            }
            set_sup_family(s, family, i) = set_sup(s, family(i))
            set_sup_contains_eq(s, family(i), x)
            if s.contains(x) {
                set_sup_contains_eq(s, set_sSup(family), x)
                u.contains(x)
            } else {
                family(i).contains(x)
                set_sSup_contains_eq(family, x)
                set_sSup(family).contains(x)
                set_sup_contains_eq(s, set_sSup(family), x)
                u.contains(x)
            }
        }
        u.contains(x) = v.contains(x)
    }
    set_ext(u, v)
}

/// Binary supremum distributes over a family supremum on the left.
theorem set_sup_sSup_distrib_right[K, I: Inhabited](family: I -> Set[K], s: Set[K]) {
    set_sup(set_sSup(family), s) = set_sSup(set_sup_family_right(family, s))
} by {
    let u = set_sup(set_sSup(family), s)
    let v = set_sSup(set_sup_family_right(family, s))
    forall(x: K) {
        if u.contains(x) {
            set_sup_contains_eq(set_sSup(family), s, x)
            if s.contains(x) {
                let i: I satisfy {
                    true
                }
                set_sup_contains_eq(family(i), s, x)
                set_sup(family(i), s).contains(x)
                set_sup_family_right(family, s, i) = set_sup(family(i), s)
                set_sup_family_right(family, s, i).contains(x)
                set_sSup_contains_eq(set_sup_family_right(family, s), x)
                v.contains(x)
            } else {
                set_sSup(family).contains(x)
                set_sSup_contains_eq(family, x)
                let i: I satisfy {
                    family(i).contains(x)
                }
                set_sup_contains_eq(family(i), s, x)
                set_sup(family(i), s).contains(x)
                set_sup_family_right(family, s, i) = set_sup(family(i), s)
                set_sup_family_right(family, s, i).contains(x)
                set_sSup_contains_eq(set_sup_family_right(family, s), x)
                v.contains(x)
            }
        }
        if v.contains(x) {
            set_sSup_contains_eq(set_sup_family_right(family, s), x)
            let i: I satisfy {
                set_sup_family_right(family, s, i).contains(x)
            }
            set_sup_family_right(family, s, i) = set_sup(family(i), s)
            set_sup_contains_eq(family(i), s, x)
            if s.contains(x) {
                set_sup_contains_eq(set_sSup(family), s, x)
                u.contains(x)
            } else {
                family(i).contains(x)
                set_sSup_contains_eq(family, x)
                set_sSup(family).contains(x)
                set_sup_contains_eq(set_sSup(family), s, x)
                u.contains(x)
            }
        }
        u.contains(x) = v.contains(x)
    }
    set_ext(u, v)
}

/// Binary infimum distributes over a family infimum on the right.
theorem set_inf_sInf_distrib_left[K, I: Inhabited](s: Set[K], family: I -> Set[K]) {
    set_inf(s, set_sInf(family)) = set_sInf(set_inf_family(s, family))
} by {
    let u = set_inf(s, set_sInf(family))
    let v = set_sInf(set_inf_family(s, family))
    forall(x: K) {
        if u.contains(x) {
            set_inf_contains_eq(s, set_sInf(family), x)
            s.contains(x)
            set_sInf(family).contains(x)
            set_sInf_contains_eq(family, x)
            forall(i: I) {
                family(i).contains(x)
                set_inf_contains_eq(s, family(i), x)
                set_inf(s, family(i)).contains(x)
                set_inf_family(s, family, i) = set_inf(s, family(i))
                set_inf_family(s, family, i).contains(x)
            }
            set_sInf_contains_eq(set_inf_family(s, family), x)
            v.contains(x)
        }
        if v.contains(x) {
            set_sInf_contains_eq(set_inf_family(s, family), x)
            let i0: I satisfy {
                true
            }
            set_inf_family(s, family, i0).contains(x)
            set_inf_family(s, family, i0) = set_inf(s, family(i0))
            set_inf_contains_eq(s, family(i0), x)
            s.contains(x)
            forall(i: I) {
                set_inf_family(s, family, i).contains(x)
                set_inf_family(s, family, i) = set_inf(s, family(i))
                set_inf_contains_eq(s, family(i), x)
                family(i).contains(x)
            }
            set_sInf_contains_eq(family, x)
            set_sInf(family).contains(x)
            set_inf_contains_eq(s, set_sInf(family), x)
            u.contains(x)
        }
        u.contains(x) = v.contains(x)
    }
    set_ext(u, v)
}

/// Binary infimum distributes over a family infimum on the left.
theorem set_inf_sInf_distrib_right[K, I: Inhabited](family: I -> Set[K], s: Set[K]) {
    set_inf(set_sInf(family), s) = set_sInf(set_inf_family_right(family, s))
} by {
    let u = set_inf(set_sInf(family), s)
    let v = set_sInf(set_inf_family_right(family, s))
    forall(x: K) {
        if u.contains(x) {
            set_inf_contains_eq(set_sInf(family), s, x)
            set_sInf(family).contains(x)
            s.contains(x)
            set_sInf_contains_eq(family, x)
            forall(i: I) {
                family(i).contains(x)
                set_inf_contains_eq(family(i), s, x)
                set_inf(family(i), s).contains(x)
                set_inf_family_right(family, s, i) = set_inf(family(i), s)
                set_inf_family_right(family, s, i).contains(x)
            }
            set_sInf_contains_eq(set_inf_family_right(family, s), x)
            v.contains(x)
        }
        if v.contains(x) {
            set_sInf_contains_eq(set_inf_family_right(family, s), x)
            let i0: I satisfy {
                true
            }
            set_inf_family_right(family, s, i0).contains(x)
            set_inf_family_right(family, s, i0) = set_inf(family(i0), s)
            set_inf_contains_eq(family(i0), s, x)
            s.contains(x)
            forall(i: I) {
                set_inf_family_right(family, s, i).contains(x)
                set_inf_family_right(family, s, i) = set_inf(family(i), s)
                set_inf_contains_eq(family(i), s, x)
                family(i).contains(x)
            }
            set_sInf_contains_eq(family, x)
            set_sInf(family).contains(x)
            set_inf_contains_eq(set_sInf(family), s, x)
            u.contains(x)
        }
        u.contains(x) = v.contains(x)
    }
    set_ext(u, v)
}

/// The supremum of the members whose indices occur in a list.
define set_list_sSup[K, I](items: List[I], family: I -> Set[K]) -> Set[K] {
    list_indexed_union(items, family)
}

/// The infimum of the members whose indices occur in a list.
define set_list_sInf[K, I](items: List[I], family: I -> Set[K]) -> Set[K] {
    list_indexed_intersection(items, family)
}

/// Membership in a list supremum is membership in some listed member.
theorem set_list_sSup_contains_eq[K, I](items: List[I], family: I -> Set[K], x: K) {
    set_list_sSup(items, family).contains(x) = exists(i: I) {
        items.contains(i) and family(i).contains(x)
    }
} by {
    list_indexed_union_contains_eq(items, family, x)
}

/// Membership in a list infimum is membership in every listed member.
theorem set_list_sInf_contains_eq[K, I](items: List[I], family: I -> Set[K], x: K) {
    set_list_sInf(items, family).contains(x) = forall(i: I) {
        items.contains(i) implies family(i).contains(x)
    }
} by {
    list_indexed_intersection_contains_eq(items, family, x)
}

/// Every listed member is contained in the list supremum.
theorem set_list_family_subset_sSup[K, I](items: List[I], family: I -> Set[K], i: I) {
    items.contains(i) implies family(i).subset(set_list_sSup(items, family))
} by {
    if items.contains(i) {
        forall(x: K) {
            if family(i).contains(x) {
                list_indexed_union_contains_of_contains(items, family, i, x)
                set_list_sSup(items, family).contains(x)
            }
        }
    }
}

/// The list supremum is contained in every common superset of the listed members.
theorem set_list_sSup_subset_of_family_subset[K, I](items: List[I], family: I -> Set[K],
    s: Set[K]) {
    (forall(i: I) { items.contains(i) implies family(i).subset(s) }) implies
    set_list_sSup(items, family).subset(s)
} by {
    if forall(i: I) { items.contains(i) implies family(i).subset(s) } {
        list_indexed_union_subset_of_family_subset(items, family, s)
        set_list_sSup(items, family).subset(s)
    }
}

/// A list supremum is contained in a set exactly when every listed member is contained in it.
theorem set_list_sSup_subset_iff[K, I](items: List[I], family: I -> Set[K], s: Set[K]) {
    set_list_sSup(items, family).subset(s) = forall(i: I) {
        items.contains(i) implies family(i).subset(s)
    }
} by {
    if set_list_sSup(items, family).subset(s) {
        forall(i: I) {
            if items.contains(i) {
                set_list_family_subset_sSup(items, family, i)
                subset_trans(family(i), set_list_sSup(items, family), s)
                family(i).subset(s)
            }
        }
    }
    if forall(i: I) { items.contains(i) implies family(i).subset(s) } {
        set_list_sSup_subset_of_family_subset(items, family, s)
        set_list_sSup(items, family).subset(s)
    }
    set_list_sSup(items, family).subset(s) = forall(i: I) {
        items.contains(i) implies family(i).subset(s)
    }
}

/// A list infimum is contained in every listed member.
theorem set_list_sInf_subset_family[K, I](items: List[I], family: I -> Set[K], i: I) {
    items.contains(i) implies set_list_sInf(items, family).subset(family(i))
} by {
    if items.contains(i) {
        forall(x: K) {
            if set_list_sInf(items, family).contains(x) {
                list_indexed_intersection_contains_at(items, family, i, x)
                family(i).contains(x)
            }
        }
    }
}

/// Every common subset of the listed members is contained in the list infimum.
theorem set_subset_list_sInf_of_subset_family[K, I](items: List[I], s: Set[K],
    family: I -> Set[K]) {
    (forall(i: I) { items.contains(i) implies s.subset(family(i)) }) implies
    s.subset(set_list_sInf(items, family))
} by {
    if forall(i: I) { items.contains(i) implies s.subset(family(i)) } {
        subset_list_indexed_intersection_of_subset_family(items, family, s)
        s.subset(set_list_sInf(items, family))
    }
}

/// A set is contained in a list infimum exactly when it is contained in every listed member.
theorem set_subset_list_sInf_iff[K, I](items: List[I], s: Set[K], family: I -> Set[K]) {
    s.subset(set_list_sInf(items, family)) = forall(i: I) {
        items.contains(i) implies s.subset(family(i))
    }
} by {
    if s.subset(set_list_sInf(items, family)) {
        forall(i: I) {
            if items.contains(i) {
                set_list_sInf_subset_family(items, family, i)
                subset_trans(s, set_list_sInf(items, family), family(i))
                s.subset(family(i))
            }
        }
    }
    if forall(i: I) { items.contains(i) implies s.subset(family(i)) } {
        set_subset_list_sInf_of_subset_family(items, s, family)
        s.subset(set_list_sInf(items, family))
    }
    s.subset(set_list_sInf(items, family)) = forall(i: I) {
        items.contains(i) implies s.subset(family(i))
    }
}

/// Pointwise inclusion of listed families preserves list suprema.
theorem set_list_sSup_monotone[K, I](items: List[I], a: I -> Set[K], b: I -> Set[K]) {
    (forall(i: I) { items.contains(i) implies a(i).subset(b(i)) }) implies
    set_list_sSup(items, a).subset(set_list_sSup(items, b))
} by {
    if forall(i: I) { items.contains(i) implies a(i).subset(b(i)) } {
        list_indexed_union_monotone(items, a, b)
        set_list_sSup(items, a) = list_indexed_union(items, a)
        set_list_sSup(items, b) = list_indexed_union(items, b)
        list_indexed_union(items, a).subset(list_indexed_union(items, b))
        set_list_sSup(items, a).subset(set_list_sSup(items, b))
    }
}

/// Pointwise inclusion of listed families preserves list infima.
theorem set_list_sInf_monotone[K, I](items: List[I], a: I -> Set[K], b: I -> Set[K]) {
    (forall(i: I) { items.contains(i) implies a(i).subset(b(i)) }) implies
    set_list_sInf(items, a).subset(set_list_sInf(items, b))
} by {
    if forall(i: I) { items.contains(i) implies a(i).subset(b(i)) } {
        list_indexed_intersection_monotone(items, a, b)
        set_list_sInf(items, a) = list_indexed_intersection(items, a)
        set_list_sInf(items, b) = list_indexed_intersection(items, b)
        list_indexed_intersection(items, a).subset(list_indexed_intersection(items, b))
        set_list_sInf(items, a).subset(set_list_sInf(items, b))
    }
}

/// Preimage commutes with list-indexed set suprema.
theorem set_preimage_list_sSup[T, U, I](f: T -> U, items: List[I], family: I -> Set[U]) {
    set_preimage(f, set_list_sSup(items, family)) =
    set_list_sSup(items, set_preimage_family(f, family))
} by {
    let u = set_preimage(f, set_list_sSup(items, family))
    let v = set_list_sSup(items, set_preimage_family(f, family))
    forall(x: T) {
        if u.contains(x) {
            set_preimage_contains_eq(f, set_list_sSup(items, family), x)
            set_list_sSup(items, family).contains(f(x))
            set_list_sSup_contains_eq(items, family, f(x))
            let i: I satisfy {
                items.contains(i) and family(i).contains(f(x))
            }
            set_preimage_contains_eq(f, family(i), x)
            set_preimage(f, family(i)).contains(x)
            set_preimage_family(f, family, i) = set_preimage(f, family(i))
            set_preimage_family(f, family, i).contains(x)
            set_list_sSup_contains_eq(items, set_preimage_family(f, family), x)
            v.contains(x)
        }
        if v.contains(x) {
            set_list_sSup_contains_eq(items, set_preimage_family(f, family), x)
            let i: I satisfy {
                items.contains(i) and set_preimage_family(f, family, i).contains(x)
            }
            set_preimage_family(f, family, i) = set_preimage(f, family(i))
            set_preimage_contains_eq(f, family(i), x)
            family(i).contains(f(x))
            set_list_sSup_contains_eq(items, family, f(x))
            set_list_sSup(items, family).contains(f(x))
            set_preimage_contains_eq(f, set_list_sSup(items, family), x)
            u.contains(x)
        }
        u.contains(x) = v.contains(x)
    }
    set_ext(u, v)
}

/// Preimage commutes with list-indexed set infima.
theorem set_preimage_list_sInf[T, U, I](f: T -> U, items: List[I], family: I -> Set[U]) {
    set_preimage(f, set_list_sInf(items, family)) =
    set_list_sInf(items, set_preimage_family(f, family))
} by {
    let u = set_preimage(f, set_list_sInf(items, family))
    let v = set_list_sInf(items, set_preimage_family(f, family))
    forall(x: T) {
        if u.contains(x) {
            set_preimage_contains_eq(f, set_list_sInf(items, family), x)
            set_list_sInf(items, family).contains(f(x))
            set_list_sInf_contains_eq(items, family, f(x))
            forall(i: I) {
                if items.contains(i) {
                    family(i).contains(f(x))
                    set_preimage_contains_eq(f, family(i), x)
                    set_preimage(f, family(i)).contains(x)
                    set_preimage_family(f, family, i).contains(x)
                }
            }
            set_list_sInf_contains_eq(items, set_preimage_family(f, family), x)
            v.contains(x)
        }
        if v.contains(x) {
            set_list_sInf_contains_eq(items, set_preimage_family(f, family), x)
            forall(i: I) {
                if items.contains(i) {
                    set_preimage_family(f, family, i).contains(x)
                    set_preimage_family(f, family, i) = set_preimage(f, family(i))
                    set_preimage_contains_eq(f, family(i), x)
                    family(i).contains(f(x))
                }
            }
            set_list_sInf_contains_eq(items, family, f(x))
            set_list_sInf(items, family).contains(f(x))
            set_preimage_contains_eq(f, set_list_sInf(items, family), x)
            u.contains(x)
        }
        u.contains(x) = v.contains(x)
    }
    set_ext(u, v)
}

/// A preimage of a list supremum is contained in a set exactly when every listed member preimage is contained in it.
theorem set_preimage_list_sSup_subset_iff[T, U, I](f: T -> U, items: List[I],
    family: I -> Set[U], s: Set[T]) {
    set_preimage(f, set_list_sSup(items, family)).subset(s) = forall(i: I) {
        items.contains(i) implies set_preimage_family(f, family, i).subset(s)
    }
} by {
    set_preimage_list_sSup(f, items, family)
    set_preimage(f, set_list_sSup(items, family)) =
        set_list_sSup(items, set_preimage_family(f, family))
    set_list_sSup_subset_iff(items, set_preimage_family(f, family), s)
    set_preimage(f, set_list_sSup(items, family)).subset(s) = forall(i: I) {
        items.contains(i) implies set_preimage_family(f, family, i).subset(s)
    }
}

/// A set is contained in a preimage of a list infimum exactly when it is contained in every listed member preimage.
theorem set_subset_preimage_list_sInf_iff[T, U, I](s: Set[T], f: T -> U, items: List[I],
    family: I -> Set[U]) {
    s.subset(set_preimage(f, set_list_sInf(items, family))) = forall(i: I) {
        items.contains(i) implies s.subset(set_preimage_family(f, family, i))
    }
} by {
    set_preimage_list_sInf(f, items, family)
    set_preimage(f, set_list_sInf(items, family)) =
        set_list_sInf(items, set_preimage_family(f, family))
    set_subset_list_sInf_iff(items, s, set_preimage_family(f, family))
    s.subset(set_preimage(f, set_list_sInf(items, family))) = forall(i: I) {
        items.contains(i) implies s.subset(set_preimage_family(f, family, i))
    }
}

/// Image commutes with list-indexed set suprema.
theorem set_image_list_sSup[T, U, I](items: List[I], family: I -> Set[T], f: T -> U) {
    set_image(set_list_sSup(items, family), f) =
    set_list_sSup(items, set_image_family(family, f))
} by {
    let u = set_image(set_list_sSup(items, family), f)
    let v = set_list_sSup(items, set_image_family(family, f))
    forall(y: U) {
        if u.contains(y) {
            set_image_contains_witness(set_list_sSup(items, family), f, y)
            let x: T satisfy {
                set_list_sSup(items, family).contains(x) and y = f(x)
            }
            set_list_sSup_contains_eq(items, family, x)
            let i: I satisfy {
                items.contains(i) and family(i).contains(x)
            }
            maps_into_set_image(family(i), f, x)
            set_image(family(i), f).contains(f(x))
            set_image_family(family, f, i) = set_image(family(i), f)
            set_image_family(family, f, i).contains(y)
            set_list_sSup_contains_eq(items, set_image_family(family, f), y)
            v.contains(y)
        }
        if v.contains(y) {
            set_list_sSup_contains_eq(items, set_image_family(family, f), y)
            let i: I satisfy {
                items.contains(i) and set_image_family(family, f, i).contains(y)
            }
            set_image_family(family, f, i) = set_image(family(i), f)
            set_image_contains_witness(family(i), f, y)
            let x: T satisfy {
                family(i).contains(x) and y = f(x)
            }
            set_list_sSup_contains_eq(items, family, x)
            set_list_sSup(items, family).contains(x)
            maps_into_set_image(set_list_sSup(items, family), f, x)
            u.contains(f(x))
            u.contains(y)
        }
        u.contains(y) = v.contains(y)
    }
    set_ext(u, v)
}

/// Image of a list supremum is contained in a target exactly when every listed member maps into it.
theorem set_image_list_sSup_subset_iff[T, U, I](items: List[I], family: I -> Set[T],
    s: Set[U], f: T -> U) {
    set_image(set_list_sSup(items, family), f).subset(s) = forall(i: I) {
        items.contains(i) implies family(i).subset(set_preimage(f, s))
    }
} by {
    set_image_subset_iff_subset_preimage(set_list_sSup(items, family), s, f)
    set_image(set_list_sSup(items, family), f).subset(s) =
        set_list_sSup(items, family).subset(set_preimage(f, s))
    set_list_sSup_subset_iff(items, family, set_preimage(f, s))
    set_image(set_list_sSup(items, family), f).subset(s) = forall(i: I) {
        items.contains(i) implies family(i).subset(set_preimage(f, s))
    }
}

/// Image of a list infimum is contained in the infimum of the listed images.
theorem set_image_list_sInf_subset[T, U, I](items: List[I], family: I -> Set[T], f: T -> U) {
    set_image(set_list_sInf(items, family), f).subset(
        set_list_sInf(items, set_image_family(family, f)))
} by {
    forall(i: I) {
        if items.contains(i) {
            set_list_sInf_subset_family(items, family, i)
            set_image_monotone(set_list_sInf(items, family), family(i), f)
            set_image(set_list_sInf(items, family), f).subset(set_image(family(i), f))
            set_image_family(family, f, i) = set_image(family(i), f)
            set_image(set_list_sInf(items, family), f).subset(set_image_family(family, f, i))
        }
    }
    set_subset_list_sInf_of_subset_family(items, set_image(set_list_sInf(items, family), f),
        set_image_family(family, f))
    set_image(set_list_sInf(items, family), f).subset(
        set_list_sInf(items, set_image_family(family, f)))
}

/// Under an injective map, a nonempty listed infimum of images is contained in the image of the listed infimum.
theorem set_image_list_sInf_reverse_of_injective[T, U, I](items: List[I],
    family: I -> Set[T], f: T -> U, i0: I) {
    is_injective_fn(f) and items.contains(i0) implies
    set_list_sInf(items, set_image_family(family, f)).subset(
        set_image(set_list_sInf(items, family), f))
} by {
    if is_injective_fn(f) and items.contains(i0) {
        forall(y: U) {
            if set_list_sInf(items, set_image_family(family, f)).contains(y) {
                set_list_sInf_contains_eq(items, set_image_family(family, f), y)
                set_image_family(family, f, i0).contains(y)
                set_image_family(family, f, i0) = set_image(family(i0), f)
                set_image_contains_witness(family(i0), f, y)
                let x: T satisfy {
                    family(i0).contains(x) and y = f(x)
                }
                forall(i: I) {
                    if items.contains(i) {
                        set_image_family(family, f, i).contains(y)
                        set_image_family(family, f, i) = set_image(family(i), f)
                        set_image_contains_witness(family(i), f, y)
                        let xi: T satisfy {
                            family(i).contains(xi) and y = f(xi)
                        }
                        f(x) = f(xi)
                        injective_fn_eq(f, x, xi)
                        x = xi
                        family(i).contains(x)
                    }
                }
                set_list_sInf_contains_eq(items, family, x)
                set_list_sInf(items, family).contains(x)
                maps_into_set_image(set_list_sInf(items, family), f, x)
                set_image(set_list_sInf(items, family), f).contains(f(x))
                set_image(set_list_sInf(items, family), f).contains(y)
            }
        }
    }
}

/// An injective map preserves nonempty listed infima by image.
theorem set_image_list_sInf_of_injective[T, U, I](items: List[I], family: I -> Set[T],
    f: T -> U, i0: I) {
    is_injective_fn(f) and items.contains(i0) implies
    set_image(set_list_sInf(items, family), f) =
        set_list_sInf(items, set_image_family(family, f))
} by {
    if is_injective_fn(f) and items.contains(i0) {
        set_image_list_sInf_subset(items, family, f)
        set_image_list_sInf_reverse_of_injective(items, family, f, i0)
        subset_antisymm(set_image(set_list_sInf(items, family), f),
            set_list_sInf(items, set_image_family(family, f)))
    }
}

/// Under a bijective map, a listed infimum of images is contained in the image of the listed infimum.
theorem set_image_list_sInf_reverse_of_bijection[T: Inhabited, U, I](items: List[I],
    family: I -> Set[T], f: T -> U) {
    is_bijection_fn(f) implies
    set_list_sInf(items, set_image_family(family, f)).subset(
        set_image(set_list_sInf(items, family), f))
} by {
    if is_bijection_fn(f) {
        bijection_fn_is_surjective(f)
        is_surjective_fn(f)
        forall(y: U) {
            if set_list_sInf(items, set_image_family(family, f)).contains(y) {
                set_list_sInf_contains_eq(items, set_image_family(family, f), y)
                forall(i: I) {
                    if items.contains(i) {
                        set_image_family(family, f, i).contains(y)
                        set_image_family(family, f, i) = set_image(family(i), f)
                        set_image_contains_inverse_of_bijection(family(i), f, y)
                        family(i).contains(inverse_fn(f, y))
                    }
                }
                set_list_sInf_contains_eq(items, family, inverse_fn(f, y))
                set_list_sInf(items, family).contains(inverse_fn(f, y))
                set_inverse_contains_imp_image_contains(set_list_sInf(items, family), f, y)
                set_image(set_list_sInf(items, family), f).contains(y)
            }
        }
    }
}

/// A bijective map preserves listed infima by image.
theorem set_image_list_sInf_of_bijection[T: Inhabited, U, I](items: List[I],
    family: I -> Set[T], f: T -> U) {
    is_bijection_fn(f) implies
    set_image(set_list_sInf(items, family), f) =
        set_list_sInf(items, set_image_family(family, f))
} by {
    if is_bijection_fn(f) {
        let u = set_image(set_list_sInf(items, family), f)
        let v = set_list_sInf(items, set_image_family(family, f))
        set_image_list_sInf_subset(items, family, f)
        u.subset(v)
        set_image_list_sInf_reverse_of_bijection(items, family, f)
        v.subset(u)
        u.subset(v) and v.subset(u)
        subset_antisymm(u, v)
        set_image(set_list_sInf(items, family), f) =
            set_list_sInf(items, set_image_family(family, f))
    }
}

/// A set is contained in an injective image of a nonempty listed infimum exactly when it is contained in every listed member image.
theorem set_subset_image_list_sInf_iff_of_injective[T, U, I](s: Set[U], items: List[I],
    family: I -> Set[T], f: T -> U, i0: I) {
    is_injective_fn(f) and items.contains(i0) implies
    s.subset(set_image(set_list_sInf(items, family), f)) = forall(i: I) {
        items.contains(i) implies s.subset(set_image_family(family, f, i))
    }
} by {
    if is_injective_fn(f) and items.contains(i0) {
        set_image_list_sInf_of_injective(items, family, f, i0)
        set_image(set_list_sInf(items, family), f) =
            set_list_sInf(items, set_image_family(family, f))
        set_subset_list_sInf_iff(items, s, set_image_family(family, f))
        s.subset(set_image(set_list_sInf(items, family), f)) = forall(i: I) {
            items.contains(i) implies s.subset(set_image_family(family, f, i))
        }
    }
}

/// A set is contained in a bijective image of a listed infimum exactly when it is contained in every listed member image.
theorem set_subset_image_list_sInf_iff_of_bijection[T: Inhabited, U, I](s: Set[U],
    items: List[I], family: I -> Set[T], f: T -> U) {
    is_bijection_fn(f) implies
    s.subset(set_image(set_list_sInf(items, family), f)) = forall(i: I) {
        items.contains(i) implies s.subset(set_image_family(family, f, i))
    }
} by {
    if is_bijection_fn(f) {
        set_image_list_sInf_of_bijection(items, family, f)
        set_image(set_list_sInf(items, family), f) =
            set_list_sInf(items, set_image_family(family, f))
        set_subset_list_sInf_iff(items, s, set_image_family(family, f))
        s.subset(set_image(set_list_sInf(items, family), f)) = forall(i: I) {
            items.contains(i) implies s.subset(set_image_family(family, f, i))
        }
    }
}

/// A list supremum is the least upper bound of its listed family.
theorem set_list_sSup_is_lub[K, I](items: List[I], family: I -> Set[K], s: Set[K]) {
    (forall(i: I) { items.contains(i) implies family(i).subset(set_list_sSup(items, family)) }) and
    (set_list_sSup(items, family).subset(s) = forall(i: I) {
        items.contains(i) implies family(i).subset(s)
    })
} by {
    forall(i: I) {
        if items.contains(i) {
            set_list_family_subset_sSup(items, family, i)
            family(i).subset(set_list_sSup(items, family))
        }
    }
    set_list_sSup_subset_iff(items, family, s)
}

/// A list infimum is the greatest lower bound of its listed family.
theorem set_list_sInf_is_glb[K, I](items: List[I], family: I -> Set[K], s: Set[K]) {
    (forall(i: I) { items.contains(i) implies set_list_sInf(items, family).subset(family(i)) }) and
    (s.subset(set_list_sInf(items, family)) = forall(i: I) {
        items.contains(i) implies s.subset(family(i))
    })
} by {
    forall(i: I) {
        if items.contains(i) {
            set_list_sInf_subset_family(items, family, i)
            set_list_sInf(items, family).subset(family(i))
        }
    }
    set_subset_list_sInf_iff(items, s, family)
}

/// The supremum indexed by the empty list is empty.
theorem set_list_sSup_nil[K, I](family: I -> Set[K]) {
    set_list_sSup(List.nil[I], family) = Set[K].empty_set
} by {
    let u = set_list_sSup(List.nil[I], family)
    forall(x: K) {
        if u.contains(x) {
            set_list_sSup_contains_eq(List.nil[I], family, x)
            let i: I satisfy {
                List.nil[I].contains(i) and family(i).contains(x)
            }
            List.nil[I].contains(i) = false
            false
        }
        empty_set_contains_eq[K](x)
        u.contains(x) = (Set[K].empty_set).contains(x)
    }
    set_ext(u, Set[K].empty_set)
}

/// The infimum indexed by the empty list is universal.
theorem set_list_sInf_nil[K, I](family: I -> Set[K]) {
    set_list_sInf(List.nil[I], family) = Set[K].universal_set
} by {
    let u = set_list_sInf(List.nil[I], family)
    forall(x: K) {
        set_list_sInf_contains_eq(List.nil[I], family, x)
        forall(i: I) {
            if List.nil[I].contains(i) {
                List.nil[I].contains(i) = false
                false
            }
        }
        u.contains(x)
        universal_set_contains_eq[K](x)
        u.contains(x) = (Set[K].universal_set).contains(x)
    }
    set_ext(u, Set[K].universal_set)
}

/// The list supremum of constantly empty sets is empty.
theorem set_list_sSup_empty_family[K, I](items: List[I]) {
    set_list_sSup(items, set_empty_family[K, I]) = Set[K].empty_set
} by {
    forall(i: I) {
        set_empty_family[K, I](i) = Set[K].empty_set
        set_empty_family[K, I](i).subset(Set[K].empty_set)
    }
    set_list_sSup_subset_of_family_subset(items, set_empty_family[K, I], Set[K].empty_set)
    set_list_sSup(items, set_empty_family[K, I]).subset(Set[K].empty_set)
    Set[K].empty_set.subset(set_list_sSup(items, set_empty_family[K, I]))
    subset_antisymm(set_list_sSup(items, set_empty_family[K, I]), Set[K].empty_set)
}

/// The list infimum of constantly universal sets is universal.
theorem set_list_sInf_universal_family[K, I](items: List[I]) {
    set_list_sInf(items, set_universal_family[K, I]) = Set[K].universal_set
} by {
    let u = set_list_sInf(items, set_universal_family[K, I])
    forall(x: K) {
        set_list_sInf_contains_eq(items, set_universal_family[K, I], x)
        forall(i: I) {
            if items.contains(i) {
                set_universal_family[K, I](i) = Set[K].universal_set
                universal_set_contains_eq[K](x)
                set_universal_family[K, I](i).contains(x)
            }
        }
        u.contains(x)
        universal_set_contains_eq[K](x)
        u.contains(x) = (Set[K].universal_set).contains(x)
    }
    set_ext(u, Set[K].universal_set)
}

/// The list supremum of a constant family is contained in the constant member.
theorem set_list_sSup_constant_family_subset[K, I](items: List[I], s: Set[K]) {
    set_list_sSup(items, set_constant_family[K, I](s)).subset(s)
} by {
    forall(i: I) {
        if items.contains(i) {
            set_constant_family(s, i) = s
            set_constant_family(s, i).subset(s)
        }
    }
    set_list_sSup_subset_of_family_subset(items, set_constant_family[K, I](s), s)
}

/// If a list contains an index, the constant member is contained in its list supremum.
theorem set_subset_list_sSup_constant_family[K, I](items: List[I], s: Set[K], i0: I) {
    items.contains(i0) implies s.subset(set_list_sSup(items, set_constant_family[K, I](s)))
} by {
    if items.contains(i0) {
        set_list_family_subset_sSup(items, set_constant_family[K, I](s), i0)
        set_constant_family(s, i0) = s
        s.subset(set_list_sSup(items, set_constant_family[K, I](s)))
    }
}

/// If a list contains an index, the list supremum of a constant family is the constant member.
theorem set_list_sSup_constant_family[K, I](items: List[I], s: Set[K], i0: I) {
    items.contains(i0) implies set_list_sSup(items, set_constant_family[K, I](s)) = s
} by {
    if items.contains(i0) {
        set_list_sSup_constant_family_subset(items, s)
        set_subset_list_sSup_constant_family(items, s, i0)
        subset_antisymm(set_list_sSup(items, set_constant_family[K, I](s)), s)
    }
}

/// The constant member is contained in the list infimum of a constant family.
theorem set_subset_list_sInf_constant_family[K, I](items: List[I], s: Set[K]) {
    s.subset(set_list_sInf(items, set_constant_family[K, I](s)))
} by {
    forall(i: I) {
        if items.contains(i) {
            set_constant_family(s, i) = s
            s.subset(set_constant_family(s, i))
        }
    }
    set_subset_list_sInf_of_subset_family(items, s, set_constant_family[K, I](s))
}

/// If a list contains an index, its list infimum of a constant family is contained in the constant member.
theorem set_list_sInf_constant_family_subset[K, I](items: List[I], s: Set[K], i0: I) {
    items.contains(i0) implies set_list_sInf(items, set_constant_family[K, I](s)).subset(s)
} by {
    if items.contains(i0) {
        set_list_sInf_subset_family(items, set_constant_family[K, I](s), i0)
        set_constant_family(s, i0) = s
        set_list_sInf(items, set_constant_family[K, I](s)).subset(s)
    }
}

/// If a list contains an index, the list infimum of a constant family is the constant member.
theorem set_list_sInf_constant_family[K, I](items: List[I], s: Set[K], i0: I) {
    items.contains(i0) implies set_list_sInf(items, set_constant_family[K, I](s)) = s
} by {
    if items.contains(i0) {
        set_list_sInf_constant_family_subset(items, s, i0)
        set_subset_list_sInf_constant_family(items, s)
        subset_antisymm(set_list_sInf(items, set_constant_family[K, I](s)), s)
    }
}

/// If a list contains an index, the list supremum of constantly universal sets is universal.
theorem set_list_sSup_universal_family[K, I](items: List[I], i0: I) {
    items.contains(i0) implies set_list_sSup(items, set_universal_family[K, I]) = Set[K].universal_set
} by {
    if items.contains(i0) {
        set_list_sSup_constant_family(items, Set[K].universal_set, i0)
        set_list_sSup(items, set_constant_family[K, I](Set[K].universal_set)) = Set[K].universal_set
        forall(x: I) {
            set_universal_family[K, I](x) = set_constant_family[K, I](Set[K].universal_set, x)
        }
        set_sSup_eq_of_family_subset(set_universal_family[K, I],
            set_constant_family[K, I](Set[K].universal_set))
        set_list_sSup(items, set_universal_family[K, I]) = Set[K].universal_set
    }
}

/// If a list contains an index, the list infimum of constantly empty sets is empty.
theorem set_list_sInf_empty_family[K, I](items: List[I], i0: I) {
    items.contains(i0) implies set_list_sInf(items, set_empty_family[K, I]) = Set[K].empty_set
} by {
    if items.contains(i0) {
        set_list_sInf_constant_family(items, Set[K].empty_set, i0)
        set_list_sInf(items, set_constant_family[K, I](Set[K].empty_set)) = Set[K].empty_set
        forall(x: I) {
            set_empty_family[K, I](x) = set_constant_family[K, I](Set[K].empty_set, x)
        }
        set_sInf_eq_of_family_subset(set_empty_family[K, I],
            set_constant_family[K, I](Set[K].empty_set))
        set_list_sInf(items, set_empty_family[K, I]) = Set[K].empty_set
    }
}


/// The list supremum is contained in the supremum of the full indexed family.
theorem set_list_sSup_subset_sSup[K, I](items: List[I], family: I -> Set[K]) {
    set_list_sSup(items, family).subset(set_sSup(family))
} by {
    list_indexed_union_subset_indexed_union(items, family)
    set_list_sSup(items, family) = list_indexed_union(items, family)
    set_sSup(family) = indexed_union(family)
    set_list_sSup(items, family).subset(set_sSup(family))
}

/// The infimum of the full indexed family is contained in the list infimum.
theorem set_sInf_subset_list_sInf[K, I](items: List[I], family: I -> Set[K]) {
    set_sInf(family).subset(set_list_sInf(items, family))
} by {
    indexed_intersection_subset_list_indexed_intersection(items, family)
    set_sInf(family) = indexed_intersection(family)
    set_list_sInf(items, family) = list_indexed_intersection(items, family)
    set_sInf(family).subset(set_list_sInf(items, family))
}

/// The supremum of the members whose natural indices are less than a bound.
define set_range_sSup[K](n: Nat, family: Nat -> Set[K]) -> Set[K] {
    range_indexed_union(n, family)
}

/// The infimum of the members whose natural indices are less than a bound.
define set_range_sInf[K](n: Nat, family: Nat -> Set[K]) -> Set[K] {
    range_indexed_intersection(n, family)
}

/// Membership in a range supremum is membership in some member below the bound.
theorem set_range_sSup_contains_eq[K](n: Nat, family: Nat -> Set[K], x: K) {
    set_range_sSup(n, family).contains(x) = exists(i: Nat) {
        i < n and family(i).contains(x)
    }
} by {
    range_indexed_union_contains_eq(n, family, x)
}

/// Membership in a range infimum is membership in every member below the bound.
theorem set_range_sInf_contains_eq[K](n: Nat, family: Nat -> Set[K], x: K) {
    set_range_sInf(n, family).contains(x) = forall(i: Nat) {
        i < n implies family(i).contains(x)
    }
} by {
    range_indexed_intersection_contains_eq(n, family, x)
}

/// Every member below the bound is contained in the range supremum.
theorem set_range_family_subset_sSup[K](n: Nat, family: Nat -> Set[K], i: Nat) {
    i < n implies family(i).subset(set_range_sSup(n, family))
} by {
    if i < n {
        forall(x: K) {
            if family(i).contains(x) {
                range_indexed_union_contains_of_lt(n, family, i, x)
                set_range_sSup(n, family).contains(x)
            }
        }
    }
}

/// The range supremum is contained in every common superset below the bound.
theorem set_range_sSup_subset_of_family_subset[K](n: Nat, family: Nat -> Set[K],
    s: Set[K]) {
    (forall(i: Nat) { i < n implies family(i).subset(s) }) implies
    set_range_sSup(n, family).subset(s)
} by {
    if forall(i: Nat) { i < n implies family(i).subset(s) } {
        range_indexed_union_subset_of_family_subset(n, family, s)
        set_range_sSup(n, family).subset(s)
    }
}

/// A range supremum is contained in a set exactly when every member below the bound is contained in it.
theorem set_range_sSup_subset_iff[K](n: Nat, family: Nat -> Set[K], s: Set[K]) {
    set_range_sSup(n, family).subset(s) = forall(i: Nat) {
        i < n implies family(i).subset(s)
    }
} by {
    if set_range_sSup(n, family).subset(s) {
        forall(i: Nat) {
            if i < n {
                set_range_family_subset_sSup(n, family, i)
                subset_trans(family(i), set_range_sSup(n, family), s)
                family(i).subset(s)
            }
        }
    }
    if forall(i: Nat) { i < n implies family(i).subset(s) } {
        set_range_sSup_subset_of_family_subset(n, family, s)
        set_range_sSup(n, family).subset(s)
    }
    set_range_sSup(n, family).subset(s) = forall(i: Nat) {
        i < n implies family(i).subset(s)
    }
}

/// A range infimum is contained in every member below the bound.
theorem set_range_sInf_subset_family[K](n: Nat, family: Nat -> Set[K], i: Nat) {
    i < n implies set_range_sInf(n, family).subset(family(i))
} by {
    if i < n {
        forall(x: K) {
            if set_range_sInf(n, family).contains(x) {
                range_indexed_intersection_contains_at(n, family, i, x)
                family(i).contains(x)
            }
        }
    }
}

/// Every common subset below the bound is contained in the range infimum.
theorem set_subset_range_sInf_of_subset_family[K](n: Nat, s: Set[K],
    family: Nat -> Set[K]) {
    (forall(i: Nat) { i < n implies s.subset(family(i)) }) implies
    s.subset(set_range_sInf(n, family))
} by {
    if forall(i: Nat) { i < n implies s.subset(family(i)) } {
        subset_range_indexed_intersection_of_subset_family(n, family, s)
        s.subset(set_range_sInf(n, family))
    }
}

/// A set is contained in a range infimum exactly when it is contained in every member below the bound.
theorem set_subset_range_sInf_iff[K](n: Nat, s: Set[K], family: Nat -> Set[K]) {
    s.subset(set_range_sInf(n, family)) = forall(i: Nat) {
        i < n implies s.subset(family(i))
    }
} by {
    if s.subset(set_range_sInf(n, family)) {
        forall(i: Nat) {
            if i < n {
                set_range_sInf_subset_family(n, family, i)
                subset_trans(s, set_range_sInf(n, family), family(i))
                s.subset(family(i))
            }
        }
    }
    if forall(i: Nat) { i < n implies s.subset(family(i)) } {
        set_subset_range_sInf_of_subset_family(n, s, family)
        s.subset(set_range_sInf(n, family))
    }
    s.subset(set_range_sInf(n, family)) = forall(i: Nat) {
        i < n implies s.subset(family(i))
    }
}

/// Pointwise inclusion below the bound preserves range suprema.
theorem set_range_sSup_monotone[K](n: Nat, a: Nat -> Set[K], b: Nat -> Set[K]) {
    (forall(i: Nat) { i < n implies a(i).subset(b(i)) }) implies
    set_range_sSup(n, a).subset(set_range_sSup(n, b))
} by {
    if forall(i: Nat) { i < n implies a(i).subset(b(i)) } {
        range_indexed_union_monotone(n, a, b)
        set_range_sSup(n, a) = range_indexed_union(n, a)
        set_range_sSup(n, b) = range_indexed_union(n, b)
        range_indexed_union(n, a).subset(range_indexed_union(n, b))
        set_range_sSup(n, a).subset(set_range_sSup(n, b))
    }
}

/// Pointwise inclusion below the bound preserves range infima.
theorem set_range_sInf_monotone[K](n: Nat, a: Nat -> Set[K], b: Nat -> Set[K]) {
    (forall(i: Nat) { i < n implies a(i).subset(b(i)) }) implies
    set_range_sInf(n, a).subset(set_range_sInf(n, b))
} by {
    if forall(i: Nat) { i < n implies a(i).subset(b(i)) } {
        range_indexed_intersection_monotone(n, a, b)
        set_range_sInf(n, a) = range_indexed_intersection(n, a)
        set_range_sInf(n, b) = range_indexed_intersection(n, b)
        range_indexed_intersection(n, a).subset(range_indexed_intersection(n, b))
        set_range_sInf(n, a).subset(set_range_sInf(n, b))
    }
}

/// Preimage commutes with bounded natural-family set suprema.
theorem set_preimage_range_sSup[T, U](f: T -> U, n: Nat, family: Nat -> Set[U]) {
    set_preimage(f, set_range_sSup(n, family)) =
    set_range_sSup(n, set_preimage_family(f, family))
} by {
    let u = set_preimage(f, set_range_sSup(n, family))
    let v = set_range_sSup(n, set_preimage_family(f, family))
    forall(x: T) {
        if u.contains(x) {
            set_preimage_contains_eq(f, set_range_sSup(n, family), x)
            set_range_sSup(n, family).contains(f(x))
            set_range_sSup_contains_eq(n, family, f(x))
            let i: Nat satisfy {
                i < n and family(i).contains(f(x))
            }
            set_preimage_contains_eq(f, family(i), x)
            set_preimage(f, family(i)).contains(x)
            set_preimage_family(f, family, i) = set_preimage(f, family(i))
            set_preimage_family(f, family, i).contains(x)
            set_range_sSup_contains_eq(n, set_preimage_family(f, family), x)
            v.contains(x)
        }
        if v.contains(x) {
            set_range_sSup_contains_eq(n, set_preimage_family(f, family), x)
            let i: Nat satisfy {
                i < n and set_preimage_family(f, family, i).contains(x)
            }
            set_preimage_family(f, family, i) = set_preimage(f, family(i))
            set_preimage_contains_eq(f, family(i), x)
            family(i).contains(f(x))
            set_range_sSup_contains_eq(n, family, f(x))
            set_range_sSup(n, family).contains(f(x))
            set_preimage_contains_eq(f, set_range_sSup(n, family), x)
            u.contains(x)
        }
        u.contains(x) = v.contains(x)
    }
    set_ext(u, v)
}

/// Preimage commutes with bounded natural-family set infima.
theorem set_preimage_range_sInf[T, U](f: T -> U, n: Nat, family: Nat -> Set[U]) {
    set_preimage(f, set_range_sInf(n, family)) =
    set_range_sInf(n, set_preimage_family(f, family))
} by {
    let u = set_preimage(f, set_range_sInf(n, family))
    let v = set_range_sInf(n, set_preimage_family(f, family))
    forall(x: T) {
        if u.contains(x) {
            set_preimage_contains_eq(f, set_range_sInf(n, family), x)
            set_range_sInf(n, family).contains(f(x))
            set_range_sInf_contains_eq(n, family, f(x))
            forall(i: Nat) {
                if i < n {
                    family(i).contains(f(x))
                    set_preimage_contains_eq(f, family(i), x)
                    set_preimage(f, family(i)).contains(x)
                    set_preimage_family(f, family, i).contains(x)
                }
            }
            set_range_sInf_contains_eq(n, set_preimage_family(f, family), x)
            v.contains(x)
        }
        if v.contains(x) {
            set_range_sInf_contains_eq(n, set_preimage_family(f, family), x)
            forall(i: Nat) {
                if i < n {
                    set_preimage_family(f, family, i).contains(x)
                    set_preimage_family(f, family, i) = set_preimage(f, family(i))
                    set_preimage_contains_eq(f, family(i), x)
                    family(i).contains(f(x))
                }
            }
            set_range_sInf_contains_eq(n, family, f(x))
            set_range_sInf(n, family).contains(f(x))
            set_preimage_contains_eq(f, set_range_sInf(n, family), x)
            u.contains(x)
        }
        u.contains(x) = v.contains(x)
    }
    set_ext(u, v)
}

/// A preimage of a bounded natural supremum is contained in a set exactly when every bounded member preimage is contained in it.
theorem set_preimage_range_sSup_subset_iff[T, U](f: T -> U, n: Nat,
    family: Nat -> Set[U], s: Set[T]) {
    set_preimage(f, set_range_sSup(n, family)).subset(s) = forall(i: Nat) {
        i < n implies set_preimage_family(f, family, i).subset(s)
    }
} by {
    set_preimage_range_sSup(f, n, family)
    set_preimage(f, set_range_sSup(n, family)) =
        set_range_sSup(n, set_preimage_family(f, family))
    set_range_sSup_subset_iff(n, set_preimage_family(f, family), s)
    set_preimage(f, set_range_sSup(n, family)).subset(s) = forall(i: Nat) {
        i < n implies set_preimage_family(f, family, i).subset(s)
    }
}

/// A set is contained in a preimage of a bounded natural infimum exactly when it is contained in every bounded member preimage.
theorem set_subset_preimage_range_sInf_iff[T, U](s: Set[T], f: T -> U, n: Nat,
    family: Nat -> Set[U]) {
    s.subset(set_preimage(f, set_range_sInf(n, family))) = forall(i: Nat) {
        i < n implies s.subset(set_preimage_family(f, family, i))
    }
} by {
    set_preimage_range_sInf(f, n, family)
    set_preimage(f, set_range_sInf(n, family)) =
        set_range_sInf(n, set_preimage_family(f, family))
    set_subset_range_sInf_iff(n, s, set_preimage_family(f, family))
    s.subset(set_preimage(f, set_range_sInf(n, family))) = forall(i: Nat) {
        i < n implies s.subset(set_preimage_family(f, family, i))
    }
}

/// Image commutes with bounded natural-family set suprema.
theorem set_image_range_sSup[T, U](n: Nat, family: Nat -> Set[T], f: T -> U) {
    set_image(set_range_sSup(n, family), f) =
    set_range_sSup(n, set_image_family(family, f))
} by {
    let u = set_image(set_range_sSup(n, family), f)
    let v = set_range_sSup(n, set_image_family(family, f))
    forall(y: U) {
        if u.contains(y) {
            set_image_contains_witness(set_range_sSup(n, family), f, y)
            let x: T satisfy {
                set_range_sSup(n, family).contains(x) and y = f(x)
            }
            set_range_sSup_contains_eq(n, family, x)
            let i: Nat satisfy {
                i < n and family(i).contains(x)
            }
            maps_into_set_image(family(i), f, x)
            set_image(family(i), f).contains(f(x))
            set_image_family(family, f, i) = set_image(family(i), f)
            set_image_family(family, f, i).contains(y)
            set_range_sSup_contains_eq(n, set_image_family(family, f), y)
            v.contains(y)
        }
        if v.contains(y) {
            set_range_sSup_contains_eq(n, set_image_family(family, f), y)
            let i: Nat satisfy {
                i < n and set_image_family(family, f, i).contains(y)
            }
            set_image_family(family, f, i) = set_image(family(i), f)
            set_image_contains_witness(family(i), f, y)
            let x: T satisfy {
                family(i).contains(x) and y = f(x)
            }
            set_range_sSup_contains_eq(n, family, x)
            set_range_sSup(n, family).contains(x)
            maps_into_set_image(set_range_sSup(n, family), f, x)
            u.contains(f(x))
            u.contains(y)
        }
        u.contains(y) = v.contains(y)
    }
    set_ext(u, v)
}

/// Image of a bounded natural supremum is contained in a target exactly when every bounded member maps into it.
theorem set_image_range_sSup_subset_iff[T, U](n: Nat, family: Nat -> Set[T],
    s: Set[U], f: T -> U) {
    set_image(set_range_sSup(n, family), f).subset(s) = forall(i: Nat) {
        i < n implies family(i).subset(set_preimage(f, s))
    }
} by {
    set_image_subset_iff_subset_preimage(set_range_sSup(n, family), s, f)
    set_image(set_range_sSup(n, family), f).subset(s) =
        set_range_sSup(n, family).subset(set_preimage(f, s))
    set_range_sSup_subset_iff(n, family, set_preimage(f, s))
    set_image(set_range_sSup(n, family), f).subset(s) = forall(i: Nat) {
        i < n implies family(i).subset(set_preimage(f, s))
    }
}

/// Image of a bounded natural infimum is contained in the infimum of the bounded images.
theorem set_image_range_sInf_subset[T, U](n: Nat, family: Nat -> Set[T], f: T -> U) {
    set_image(set_range_sInf(n, family), f).subset(
        set_range_sInf(n, set_image_family(family, f)))
} by {
    forall(i: Nat) {
        if i < n {
            set_range_sInf_subset_family(n, family, i)
            set_image_monotone(set_range_sInf(n, family), family(i), f)
            set_image(set_range_sInf(n, family), f).subset(set_image(family(i), f))
            set_image_family(family, f, i) = set_image(family(i), f)
            set_image(set_range_sInf(n, family), f).subset(set_image_family(family, f, i))
        }
    }
    set_subset_range_sInf_of_subset_family(n, set_image(set_range_sInf(n, family), f),
        set_image_family(family, f))
    set_image(set_range_sInf(n, family), f).subset(
        set_range_sInf(n, set_image_family(family, f)))
}

/// Under an injective map, a nonempty bounded infimum of images is contained in the image of the bounded infimum.
theorem set_image_range_sInf_reverse_of_injective[T, U](n: Nat, family: Nat -> Set[T],
    f: T -> U, i0: Nat) {
    is_injective_fn(f) and i0 < n implies
    set_range_sInf(n, set_image_family(family, f)).subset(
        set_image(set_range_sInf(n, family), f))
} by {
    if is_injective_fn(f) and i0 < n {
        forall(y: U) {
            if set_range_sInf(n, set_image_family(family, f)).contains(y) {
                set_range_sInf_contains_eq(n, set_image_family(family, f), y)
                set_image_family(family, f, i0).contains(y)
                set_image_family(family, f, i0) = set_image(family(i0), f)
                set_image_contains_witness(family(i0), f, y)
                let x: T satisfy {
                    family(i0).contains(x) and y = f(x)
                }
                forall(i: Nat) {
                    if i < n {
                        set_image_family(family, f, i).contains(y)
                        set_image_family(family, f, i) = set_image(family(i), f)
                        set_image_contains_witness(family(i), f, y)
                        let xi: T satisfy {
                            family(i).contains(xi) and y = f(xi)
                        }
                        f(x) = f(xi)
                        injective_fn_eq(f, x, xi)
                        x = xi
                        family(i).contains(x)
                    }
                }
                set_range_sInf_contains_eq(n, family, x)
                set_range_sInf(n, family).contains(x)
                maps_into_set_image(set_range_sInf(n, family), f, x)
                set_image(set_range_sInf(n, family), f).contains(f(x))
                set_image(set_range_sInf(n, family), f).contains(y)
            }
        }
    }
}

/// An injective map preserves nonempty bounded infima by image.
theorem set_image_range_sInf_of_injective[T, U](n: Nat, family: Nat -> Set[T],
    f: T -> U, i0: Nat) {
    is_injective_fn(f) and i0 < n implies
    set_image(set_range_sInf(n, family), f) =
        set_range_sInf(n, set_image_family(family, f))
} by {
    if is_injective_fn(f) and i0 < n {
        let u = set_image(set_range_sInf(n, family), f)
        let v = set_range_sInf(n, set_image_family(family, f))
        set_image_range_sInf_subset(n, family, f)
        u.subset(v)
        set_image_range_sInf_reverse_of_injective(n, family, f, i0)
        v.subset(u)
        u.subset(v) and v.subset(u)
        subset_antisymm(u, v)
        set_image(set_range_sInf(n, family), f) =
            set_range_sInf(n, set_image_family(family, f))
    }
}

/// Under a bijective map, a bounded infimum of images is contained in the image of the bounded infimum.
theorem set_image_range_sInf_reverse_of_bijection[T: Inhabited, U](n: Nat,
    family: Nat -> Set[T], f: T -> U) {
    is_bijection_fn(f) implies
    set_range_sInf(n, set_image_family(family, f)).subset(
        set_image(set_range_sInf(n, family), f))
} by {
    if is_bijection_fn(f) {
        bijection_fn_is_surjective(f)
        is_surjective_fn(f)
        forall(y: U) {
            if set_range_sInf(n, set_image_family(family, f)).contains(y) {
                set_range_sInf_contains_eq(n, set_image_family(family, f), y)
                forall(i: Nat) {
                    if i < n {
                        set_image_family(family, f, i).contains(y)
                        set_image_family(family, f, i) = set_image(family(i), f)
                        set_image_contains_inverse_of_bijection(family(i), f, y)
                        family(i).contains(inverse_fn(f, y))
                    }
                }
                set_range_sInf_contains_eq(n, family, inverse_fn(f, y))
                set_range_sInf(n, family).contains(inverse_fn(f, y))
                set_inverse_contains_imp_image_contains(set_range_sInf(n, family), f, y)
                set_image(set_range_sInf(n, family), f).contains(y)
            }
        }
    }
}

/// A bijective map preserves bounded natural-family infima by image.
theorem set_image_range_sInf_of_bijection[T: Inhabited, U](n: Nat, family: Nat -> Set[T],
    f: T -> U) {
    is_bijection_fn(f) implies
    set_image(set_range_sInf(n, family), f) =
        set_range_sInf(n, set_image_family(family, f))
} by {
    if is_bijection_fn(f) {
        let u = set_image(set_range_sInf(n, family), f)
        let v = set_range_sInf(n, set_image_family(family, f))
        set_image_range_sInf_subset(n, family, f)
        u.subset(v)
        set_image_range_sInf_reverse_of_bijection(n, family, f)
        v.subset(u)
        u.subset(v) and v.subset(u)
        subset_antisymm(u, v)
        set_image(set_range_sInf(n, family), f) =
            set_range_sInf(n, set_image_family(family, f))
    }
}

/// A set is contained in an injective image of a nonempty bounded infimum exactly when it is contained in every bounded member image.
theorem set_subset_image_range_sInf_iff_of_injective[T, U](s: Set[U], n: Nat,
    family: Nat -> Set[T], f: T -> U, i0: Nat) {
    is_injective_fn(f) and i0 < n implies
    s.subset(set_image(set_range_sInf(n, family), f)) = forall(i: Nat) {
        i < n implies s.subset(set_image_family(family, f, i))
    }
} by {
    if is_injective_fn(f) and i0 < n {
        set_image_range_sInf_of_injective(n, family, f, i0)
        set_image(set_range_sInf(n, family), f) =
            set_range_sInf(n, set_image_family(family, f))
        set_subset_range_sInf_iff(n, s, set_image_family(family, f))
        s.subset(set_image(set_range_sInf(n, family), f)) = forall(i: Nat) {
            i < n implies s.subset(set_image_family(family, f, i))
        }
    }
}

/// A set is contained in a bijective image of a bounded infimum exactly when it is contained in every bounded member image.
theorem set_subset_image_range_sInf_iff_of_bijection[T: Inhabited, U](s: Set[U],
    n: Nat, family: Nat -> Set[T], f: T -> U) {
    is_bijection_fn(f) implies
    s.subset(set_image(set_range_sInf(n, family), f)) = forall(i: Nat) {
        i < n implies s.subset(set_image_family(family, f, i))
    }
} by {
    if is_bijection_fn(f) {
        set_image_range_sInf_of_bijection(n, family, f)
        set_image(set_range_sInf(n, family), f) =
            set_range_sInf(n, set_image_family(family, f))
        set_subset_range_sInf_iff(n, s, set_image_family(family, f))
        s.subset(set_image(set_range_sInf(n, family), f)) = forall(i: Nat) {
            i < n implies s.subset(set_image_family(family, f, i))
        }
    }
}

/// A range supremum is the least upper bound of the bounded family.
theorem set_range_sSup_is_lub[K](n: Nat, family: Nat -> Set[K], s: Set[K]) {
    (forall(i: Nat) { i < n implies family(i).subset(set_range_sSup(n, family)) }) and
    (set_range_sSup(n, family).subset(s) = forall(i: Nat) {
        i < n implies family(i).subset(s)
    })
} by {
    forall(i: Nat) {
        if i < n {
            set_range_family_subset_sSup(n, family, i)
            family(i).subset(set_range_sSup(n, family))
        }
    }
    set_range_sSup_subset_iff(n, family, s)
}

/// A range infimum is the greatest lower bound of the bounded family.
theorem set_range_sInf_is_glb[K](n: Nat, family: Nat -> Set[K], s: Set[K]) {
    (forall(i: Nat) { i < n implies set_range_sInf(n, family).subset(family(i)) }) and
    (s.subset(set_range_sInf(n, family)) = forall(i: Nat) {
        i < n implies s.subset(family(i))
    })
} by {
    forall(i: Nat) {
        if i < n {
            set_range_sInf_subset_family(n, family, i)
            set_range_sInf(n, family).subset(family(i))
        }
    }
    set_subset_range_sInf_iff(n, s, family)
}

/// The supremum over an empty range is empty.
theorem set_range_sSup_zero[K](family: Nat -> Set[K]) {
    set_range_sSup(Nat.0, family) = Set[K].empty_set
} by {
    range_indexed_union_zero(family)
}

/// The infimum over an empty range is universal.
theorem set_range_sInf_zero[K](family: Nat -> Set[K]) {
    set_range_sInf(Nat.0, family) = Set[K].universal_set
} by {
    range_indexed_intersection_zero(family)
}

/// The bounded supremum of constantly empty sets is empty.
theorem set_range_sSup_empty_family[K](n: Nat) {
    set_range_sSup(n, set_empty_family[K, Nat]) = Set[K].empty_set
} by {
    forall(i: Nat) {
        if i < n {
            set_empty_family[K, Nat](i) = Set[K].empty_set
            set_empty_family[K, Nat](i).subset(Set[K].empty_set)
        }
    }
    set_range_sSup_subset_of_family_subset(n, set_empty_family[K, Nat], Set[K].empty_set)
    set_range_sSup(n, set_empty_family[K, Nat]).subset(Set[K].empty_set)
    Set[K].empty_set.subset(set_range_sSup(n, set_empty_family[K, Nat]))
    subset_antisymm(set_range_sSup(n, set_empty_family[K, Nat]), Set[K].empty_set)
}

/// The bounded infimum of constantly universal sets is universal.
theorem set_range_sInf_universal_family[K](n: Nat) {
    set_range_sInf(n, set_universal_family[K, Nat]) = Set[K].universal_set
} by {
    let u = set_range_sInf(n, set_universal_family[K, Nat])
    forall(x: K) {
        set_range_sInf_contains_eq(n, set_universal_family[K, Nat], x)
        forall(i: Nat) {
            if i < n {
                set_universal_family[K, Nat](i) = Set[K].universal_set
                universal_set_contains_eq[K](x)
                set_universal_family[K, Nat](i).contains(x)
            }
        }
        u.contains(x)
        universal_set_contains_eq[K](x)
        u.contains(x) = (Set[K].universal_set).contains(x)
    }
    set_ext(u, Set[K].universal_set)
}

/// The bounded supremum of a constant family is contained in the constant member.
theorem set_range_sSup_constant_family_subset[K](n: Nat, s: Set[K]) {
    set_range_sSup(n, set_constant_family[K, Nat](s)).subset(s)
} by {
    forall(i: Nat) {
        if i < n {
            set_constant_family(s, i) = s
            set_constant_family(s, i).subset(s)
        }
    }
    set_range_sSup_subset_of_family_subset(n, set_constant_family[K, Nat](s), s)
}

/// If a natural index lies below the bound, the constant member is contained in the bounded supremum.
theorem set_subset_range_sSup_constant_family[K](n: Nat, s: Set[K], i0: Nat) {
    i0 < n implies s.subset(set_range_sSup(n, set_constant_family[K, Nat](s)))
} by {
    if i0 < n {
        set_range_family_subset_sSup(n, set_constant_family[K, Nat](s), i0)
        set_constant_family(s, i0) = s
        s.subset(set_range_sSup(n, set_constant_family[K, Nat](s)))
    }
}

/// If a natural index lies below the bound, the bounded supremum of a constant family is the constant member.
theorem set_range_sSup_constant_family[K](n: Nat, s: Set[K], i0: Nat) {
    i0 < n implies set_range_sSup(n, set_constant_family[K, Nat](s)) = s
} by {
    if i0 < n {
        set_range_sSup_constant_family_subset(n, s)
        set_subset_range_sSup_constant_family(n, s, i0)
        subset_antisymm(set_range_sSup(n, set_constant_family[K, Nat](s)), s)
    }
}

/// The constant member is contained in the bounded infimum of a constant family.
theorem set_subset_range_sInf_constant_family[K](n: Nat, s: Set[K]) {
    s.subset(set_range_sInf(n, set_constant_family[K, Nat](s)))
} by {
    forall(i: Nat) {
        if i < n {
            set_constant_family(s, i) = s
            s.subset(set_constant_family(s, i))
        }
    }
    set_subset_range_sInf_of_subset_family(n, s, set_constant_family[K, Nat](s))
}

/// If a natural index lies below the bound, the bounded infimum of a constant family is contained in the constant member.
theorem set_range_sInf_constant_family_subset[K](n: Nat, s: Set[K], i0: Nat) {
    i0 < n implies set_range_sInf(n, set_constant_family[K, Nat](s)).subset(s)
} by {
    if i0 < n {
        set_range_sInf_subset_family(n, set_constant_family[K, Nat](s), i0)
        set_constant_family(s, i0) = s
        set_range_sInf(n, set_constant_family[K, Nat](s)).subset(s)
    }
}

/// If a natural index lies below the bound, the bounded infimum of a constant family is the constant member.
theorem set_range_sInf_constant_family[K](n: Nat, s: Set[K], i0: Nat) {
    i0 < n implies set_range_sInf(n, set_constant_family[K, Nat](s)) = s
} by {
    if i0 < n {
        set_range_sInf_constant_family_subset(n, s, i0)
        set_subset_range_sInf_constant_family(n, s)
        subset_antisymm(set_range_sInf(n, set_constant_family[K, Nat](s)), s)
    }
}

/// If a natural index lies below the bound, the bounded supremum of constantly universal sets is universal.
theorem set_range_sSup_universal_family[K](n: Nat, i0: Nat) {
    i0 < n implies set_range_sSup(n, set_universal_family[K, Nat]) = Set[K].universal_set
} by {
    if i0 < n {
        set_range_sSup_constant_family(n, Set[K].universal_set, i0)
        set_range_sSup(n, set_constant_family[K, Nat](Set[K].universal_set)) = Set[K].universal_set
        forall(x: Nat) {
            set_universal_family[K, Nat](x) = set_constant_family[K, Nat](Set[K].universal_set, x)
        }
        set_sSup_eq_of_family_subset(set_universal_family[K, Nat],
            set_constant_family[K, Nat](Set[K].universal_set))
        set_range_sSup(n, set_universal_family[K, Nat]) = Set[K].universal_set
    }
}

/// If a natural index lies below the bound, the bounded infimum of constantly empty sets is empty.
theorem set_range_sInf_empty_family[K](n: Nat, i0: Nat) {
    i0 < n implies set_range_sInf(n, set_empty_family[K, Nat]) = Set[K].empty_set
} by {
    if i0 < n {
        set_range_sInf_constant_family(n, Set[K].empty_set, i0)
        set_range_sInf(n, set_constant_family[K, Nat](Set[K].empty_set)) = Set[K].empty_set
        forall(x: Nat) {
            set_empty_family[K, Nat](x) = set_constant_family[K, Nat](Set[K].empty_set, x)
        }
        set_sInf_eq_of_family_subset(set_empty_family[K, Nat],
            set_constant_family[K, Nat](Set[K].empty_set))
        set_range_sInf(n, set_empty_family[K, Nat]) = Set[K].empty_set
    }
}

/// The supremum over a successor range adds the last member by binary supremum.
theorem set_range_sSup_suc_eq_sup[K](n: Nat, family: Nat -> Set[K]) {
    set_range_sSup(n.suc, family) = set_sup(set_range_sSup(n, family), family(n))
} by {
    range_indexed_union_suc_eq_union(n, family)
}

/// The infimum over a successor range adds the last member by binary infimum.
theorem set_range_sInf_suc_eq_inf[K](n: Nat, family: Nat -> Set[K]) {
    set_range_sInf(n.suc, family) = set_inf(set_range_sInf(n, family), family(n))
} by {
    range_indexed_intersection_suc_eq_intersection(n, family)
}

/// The range supremum is contained in the supremum of the full natural family.
theorem set_range_sSup_subset_sSup[K](n: Nat, family: Nat -> Set[K]) {
    set_range_sSup(n, family).subset(set_sSup(family))
} by {
    forall(x: K) {
        if set_range_sSup(n, family).contains(x) {
            set_range_sSup_contains_eq(n, family, x)
            let i: Nat satisfy {
                i < n and family(i).contains(x)
            }
            set_sSup_contains_eq(family, x)
            set_sSup(family).contains(x)
        }
    }
}

/// The infimum of the full natural family is contained in the range infimum.
theorem set_sInf_subset_range_sInf[K](n: Nat, family: Nat -> Set[K]) {
    set_sInf(family).subset(set_range_sInf(n, family))
} by {
    set_subset_range_sInf_of_subset_family(n, set_sInf(family), family)
    set_sInf(family).subset(set_range_sInf(n, family))
}

/// Enlarging the bound can only enlarge a range supremum.
theorem set_range_sSup_subset_of_bound_le[K](m: Nat, n: Nat, family: Nat -> Set[K]) {
    m <= n implies set_range_sSup(m, family).subset(set_range_sSup(n, family))
} by {
    if m <= n {
        range_indexed_union_subset_of_bound_le(m, n, family)
        range_indexed_union(m, family).subset(range_indexed_union(n, family))
        set_range_sSup(m, family).subset(set_range_sSup(n, family))
    }
}

/// Enlarging the bound can only shrink a range infimum.
theorem set_range_sInf_subset_of_bound_le[K](m: Nat, n: Nat, family: Nat -> Set[K]) {
    m <= n implies set_range_sInf(n, family).subset(set_range_sInf(m, family))
} by {
    if m <= n {
        range_indexed_intersection_subset_of_bound_le(m, n, family)
        range_indexed_intersection(n, family).subset(range_indexed_intersection(m, family))
        set_range_sInf(n, family).subset(set_range_sInf(m, family))
    }
}

/// The supremum of bounded complements is the complement of the bounded infimum.
theorem set_range_sSup_complement_eq_sInf_complement[K](n: Nat, family: Nat -> Set[K]) {
    set_range_sSup(n, function(i: Nat) { family(i).c }) = set_range_sInf(n, family).c
} by {
    range_indexed_union_complement_eq_intersection_complement(n, family)
    range_indexed_union(n, function(i: Nat) { family(i).c }) =
        range_indexed_intersection(n, family).c
    set_range_sSup(n, function(i: Nat) { family(i).c }) = set_range_sInf(n, family).c
}

/// The infimum of bounded complements is the complement of the bounded supremum.
theorem set_range_sInf_complement_eq_sSup_complement[K](n: Nat, family: Nat -> Set[K]) {
    set_range_sInf(n, function(i: Nat) { family(i).c }) = set_range_sSup(n, family).c
} by {
    range_indexed_intersection_complement_eq_union_complement(n, family)
    range_indexed_intersection(n, function(i: Nat) { family(i).c }) =
        range_indexed_union(n, family).c
    set_range_sInf(n, function(i: Nat) { family(i).c }) = set_range_sSup(n, family).c
}

/// The supremum of an initial segment of an increasing family is contained in its last member.
theorem set_range_sSup_suc_subset_of_increasing[K](n: Nat, family: Nat -> Set[K]) {
    is_increasing(family) implies set_range_sSup(n.suc, family).subset(family(n))
} by {
    if is_increasing(family) {
        range_indexed_union_suc_subset_of_increasing(n, family)
        range_indexed_union(n.suc, family).subset(family(n))
        set_range_sSup(n.suc, family).subset(family(n))
    }
}

/// The last member of an initial segment is contained in its supremum.
theorem set_family_subset_range_sSup_suc[K](n: Nat, family: Nat -> Set[K]) {
    family(n).subset(set_range_sSup(n.suc, family))
} by {
    family_subset_range_indexed_union_suc(n, family)
    family(n).subset(range_indexed_union(n.suc, family))
    family(n).subset(set_range_sSup(n.suc, family))
}

/// The supremum of an initial segment of an increasing family is its last member.
theorem set_range_sSup_suc_of_increasing[K](n: Nat, family: Nat -> Set[K]) {
    is_increasing(family) implies set_range_sSup(n.suc, family) = family(n)
} by {
    if is_increasing(family) {
        range_indexed_union_suc_of_increasing(n, family)
        range_indexed_union(n.suc, family) = family(n)
        set_range_sSup(n.suc, family) = family(n)
    }
}

/// A decreasing family's last member is contained in the infimum of its initial segment.
theorem set_decreasing_subset_range_sInf_suc[K](n: Nat, family: Nat -> Set[K]) {
    is_decreasing(family) implies family(n).subset(set_range_sInf(n.suc, family))
} by {
    if is_decreasing(family) {
        decreasing_subset_range_indexed_intersection_suc(n, family)
        family(n).subset(range_indexed_intersection(n.suc, family))
        family(n).subset(set_range_sInf(n.suc, family))
    }
}

/// The infimum of an initial segment is contained in its last member.
theorem set_range_sInf_suc_subset_family[K](n: Nat, family: Nat -> Set[K]) {
    set_range_sInf(n.suc, family).subset(family(n))
} by {
    range_indexed_intersection_suc_subset_family(n, family)
    range_indexed_intersection(n.suc, family).subset(family(n))
    set_range_sInf(n.suc, family).subset(family(n))
}

/// The infimum of an initial segment of a decreasing family is its last member.
theorem set_range_sInf_suc_of_decreasing[K](n: Nat, family: Nat -> Set[K]) {
    is_decreasing(family) implies set_range_sInf(n.suc, family) = family(n)
} by {
    if is_decreasing(family) {
        range_indexed_intersection_suc_of_decreasing(n, family)
        range_indexed_intersection(n.suc, family) = family(n)
        set_range_sInf(n.suc, family) = family(n)
    }
}

/// A bounded supremum is contained in the sequential supremum.
theorem set_range_sSup_subset_seq_union[K](n: Nat, family: Nat -> Set[K]) {
    set_range_sSup(n, family).subset(seq_union(family))
} by {
    range_indexed_union_subset_seq_union(n, family)
    range_indexed_union(n, family).subset(seq_union(family))
    set_range_sSup(n, family).subset(seq_union(family))
}

/// The sequential infimum is contained in every bounded infimum.
theorem set_seq_intersection_subset_range_sInf[K](n: Nat, family: Nat -> Set[K]) {
    seq_intersection(family).subset(set_range_sInf(n, family))
} by {
    seq_intersection_subset_range_indexed_intersection(n, family)
    seq_intersection(family).subset(range_indexed_intersection(n, family))
    seq_intersection(family).subset(set_range_sInf(n, family))
}

/// The generic natural-family supremum is the sequence union.
theorem set_sSup_eq_seq_union[K](family: Nat -> Set[K]) {
    set_sSup(family) = seq_union(family)
} by {
    let u = set_sSup(family)
    let v = seq_union(family)
    forall(x: K) {
        set_sSup_contains_eq(family, x)
        seq_union_contains_eq(family, x)
        u.contains(x) = v.contains(x)
    }
    set_ext(u, v)
}

/// The generic natural-family infimum is the sequence intersection.
theorem set_sInf_eq_seq_intersection[K](family: Nat -> Set[K]) {
    set_sInf(family) = seq_intersection(family)
} by {
    let u = set_sInf(family)
    let v = seq_intersection(family)
    forall(x: K) {
        set_sInf_contains_eq(family, x)
        seq_intersection_contains_eq(family, x)
        u.contains(x) = v.contains(x)
    }
    set_ext(u, v)
}

/// A sequence union is the least upper bound of the sequence.
theorem set_seq_union_is_lub[K](family: Nat -> Set[K], s: Set[K]) {
    (forall(i: Nat) { family(i).subset(seq_union(family)) }) and
    (seq_union(family).subset(s) = forall(i: Nat) { family(i).subset(s) })
} by {
    set_sSup_eq_seq_union(family)
    set_sSup_is_lub(family, s)
    forall(i: Nat) {
        family(i).subset(set_sSup(family))
        family(i).subset(seq_union(family))
    }
    set_sSup(family).subset(s) = forall(i: Nat) { family(i).subset(s) }
    seq_union(family).subset(s) = forall(i: Nat) { family(i).subset(s) }
}

/// A sequence intersection is the greatest lower bound of the sequence.
theorem set_seq_intersection_is_glb[K](family: Nat -> Set[K], s: Set[K]) {
    (forall(i: Nat) { seq_intersection(family).subset(family(i)) }) and
    (s.subset(seq_intersection(family)) = forall(i: Nat) { s.subset(family(i)) })
} by {
    set_sInf_eq_seq_intersection(family)
    set_sInf_is_glb(family, s)
    forall(i: Nat) {
        set_sInf(family).subset(family(i))
        seq_intersection(family).subset(family(i))
    }
    s.subset(set_sInf(family)) = forall(i: Nat) { s.subset(family(i)) }
    s.subset(seq_intersection(family)) = forall(i: Nat) { s.subset(family(i)) }
}

/// A sequence union is contained in a set exactly when every term is contained in it.
theorem set_seq_union_subset_iff[K](family: Nat -> Set[K], s: Set[K]) {
    seq_union(family).subset(s) = forall(i: Nat) { family(i).subset(s) }
} by {
    set_sSup_eq_seq_union(family)
    set_sSup_subset_iff(family, s)
    seq_union(family).subset(s) = forall(i: Nat) { family(i).subset(s) }
}

/// A set is contained in a sequence intersection exactly when it is contained in every term.
theorem set_subset_seq_intersection_iff[K](s: Set[K], family: Nat -> Set[K]) {
    s.subset(seq_intersection(family)) = forall(i: Nat) { s.subset(family(i)) }
} by {
    set_sInf_eq_seq_intersection(family)
    set_subset_sInf_iff(s, family)
    s.subset(seq_intersection(family)) = forall(i: Nat) { s.subset(family(i)) }
}

/// Pointwise inclusion of sequences preserves sequence unions.
theorem set_seq_union_monotone[K](a: Nat -> Set[K], b: Nat -> Set[K]) {
    (forall(i: Nat) { a(i).subset(b(i)) }) implies seq_union(a).subset(seq_union(b))
} by {
    if forall(i: Nat) { a(i).subset(b(i)) } {
        set_sSup_monotone(a, b)
        set_sSup_eq_seq_union(a)
        set_sSup_eq_seq_union(b)
        set_sSup(a).subset(set_sSup(b))
        seq_union(a).subset(seq_union(b))
    }
}

/// Pointwise inclusion of sequences preserves sequence intersections.
theorem set_seq_intersection_monotone[K](a: Nat -> Set[K], b: Nat -> Set[K]) {
    (forall(i: Nat) { a(i).subset(b(i)) }) implies
    seq_intersection(a).subset(seq_intersection(b))
} by {
    if forall(i: Nat) { a(i).subset(b(i)) } {
        set_sInf_monotone(a, b)
        set_sInf_eq_seq_intersection(a)
        set_sInf_eq_seq_intersection(b)
        set_sInf(a).subset(set_sInf(b))
        seq_intersection(a).subset(seq_intersection(b))
    }
}

/// Preimage commutes with sequence unions.
theorem set_preimage_seq_union[T, U](f: T -> U, family: Nat -> Set[U]) {
    set_preimage(f, seq_union(family)) = seq_union(set_preimage_family(f, family))
} by {
    set_sSup_eq_seq_union(family)
    set_sSup_eq_seq_union(set_preimage_family(f, family))
    set_preimage_sSup(f, family)
    set_preimage(f, set_sSup(family)) = set_sSup(set_preimage_family(f, family))
    set_preimage(f, seq_union(family)) = seq_union(set_preimage_family(f, family))
}

/// Preimage commutes with sequence intersections.
theorem set_preimage_seq_intersection[T, U](f: T -> U, family: Nat -> Set[U]) {
    set_preimage(f, seq_intersection(family)) =
    seq_intersection(set_preimage_family(f, family))
} by {
    set_sInf_eq_seq_intersection(family)
    set_sInf_eq_seq_intersection(set_preimage_family(f, family))
    set_preimage_sInf(f, family)
    set_preimage(f, set_sInf(family)) = set_sInf(set_preimage_family(f, family))
    set_preimage(f, seq_intersection(family)) =
        seq_intersection(set_preimage_family(f, family))
}

/// A preimage of a sequence union is contained in a set exactly when every term preimage is contained in it.
theorem set_preimage_seq_union_subset_iff[T, U](f: T -> U, family: Nat -> Set[U],
    s: Set[T]) {
    set_preimage(f, seq_union(family)).subset(s) = forall(i: Nat) {
        set_preimage_family(f, family, i).subset(s)
    }
} by {
    set_preimage_seq_union(f, family)
    set_seq_union_subset_iff(set_preimage_family(f, family), s)
    set_preimage(f, seq_union(family)).subset(s) = forall(i: Nat) {
        set_preimage_family(f, family, i).subset(s)
    }
}

/// A set is contained in a preimage of a sequence intersection exactly when it is contained in every term preimage.
theorem set_subset_preimage_seq_intersection_iff[T, U](s: Set[T], f: T -> U,
    family: Nat -> Set[U]) {
    s.subset(set_preimage(f, seq_intersection(family))) = forall(i: Nat) {
        s.subset(set_preimage_family(f, family, i))
    }
} by {
    set_preimage_seq_intersection(f, family)
    set_subset_seq_intersection_iff(s, set_preimage_family(f, family))
    s.subset(set_preimage(f, seq_intersection(family))) = forall(i: Nat) {
        s.subset(set_preimage_family(f, family, i))
    }
}

/// The image of a sequence union is the sequence union of the images.
theorem set_image_seq_union[T, U](family: Nat -> Set[T], f: T -> U) {
    set_image(seq_union(family), f) = seq_union(set_image_family(family, f))
} by {
    seq_union_set_image(family, f)
    seq_union(set_image_seq(family, f)) = set_image(seq_union(family), f)
    forall(i: Nat) {
        set_image_seq(family, f, i) = set_image_family(family, f, i)
    }
    set_sSup_eq_of_family_subset(set_image_seq(family, f), set_image_family(family, f))
    set_sSup(set_image_seq(family, f)) = set_sSup(set_image_family(family, f))
    set_sSup_eq_seq_union(set_image_seq(family, f))
    set_sSup_eq_seq_union(set_image_family(family, f))
    seq_union(set_image_seq(family, f)) = seq_union(set_image_family(family, f))
    set_image(seq_union(family), f) = seq_union(set_image_family(family, f))
}

/// The image of a sequence intersection is contained in the sequence intersection of the images.
theorem set_image_seq_intersection_subset[T, U](family: Nat -> Set[T], f: T -> U) {
    set_image(seq_intersection(family), f).subset(seq_intersection(set_image_family(family, f)))
} by {
    set_sInf_eq_seq_intersection(family)
    set_sInf_eq_seq_intersection(set_image_family(family, f))
    set_image_sInf_subset(family, f)
    set_image(set_sInf(family), f).subset(set_sInf(set_image_family(family, f)))
    set_image(seq_intersection(family), f).subset(seq_intersection(set_image_family(family, f)))
}

/// Under an injective map, images preserve sequence intersections.
theorem set_image_seq_intersection_of_injective[T, U](family: Nat -> Set[T], f: T -> U) {
    is_injective_fn(f) implies
    set_image(seq_intersection(family), f) = seq_intersection(set_image_family(family, f))
} by {
    if is_injective_fn(f) {
        set_sInf_eq_seq_intersection(family)
        set_sInf_eq_seq_intersection(set_image_family(family, f))
        set_image_sInf_of_injective(family, f)
        set_image(set_sInf(family), f) = set_sInf(set_image_family(family, f))
        set_image(seq_intersection(family), f) =
            seq_intersection(set_image_family(family, f))
    }
}

/// Under a bijective map, images preserve sequence intersections.
theorem set_image_seq_intersection_of_bijection[T: Inhabited, U](family: Nat -> Set[T],
    f: T -> U) {
    is_bijection_fn(f) implies
    set_image(seq_intersection(family), f) = seq_intersection(set_image_family(family, f))
} by {
    if is_bijection_fn(f) {
        set_sInf_eq_seq_intersection(family)
        set_sInf_eq_seq_intersection(set_image_family(family, f))
        set_image_sInf_of_bijection(family, f)
        set_image(set_sInf(family), f) = set_sInf(set_image_family(family, f))
        set_image(seq_intersection(family), f) =
            seq_intersection(set_image_family(family, f))
    }
}

/// Image of a sequence union is contained in a target exactly when every term maps into it.
theorem set_image_seq_union_subset_iff[T, U](family: Nat -> Set[T], s: Set[U],
    f: T -> U) {
    set_image(seq_union(family), f).subset(s) = forall(i: Nat) {
        family(i).subset(set_preimage(f, s))
    }
} by {
    set_image_seq_union(family, f)
    set_seq_union_subset_iff(set_image_family(family, f), s)
    set_image_subset_iff_subset_preimage(seq_union(family), s, f)
    set_seq_union_subset_iff(family, set_preimage(f, s))
    set_image(seq_union(family), f).subset(s) = forall(i: Nat) {
        family(i).subset(set_preimage(f, s))
    }
}

/// A set is contained in an injective image of a sequence intersection exactly when it is contained in every term image.
theorem set_subset_image_seq_intersection_iff_of_injective[T, U](s: Set[U],
    family: Nat -> Set[T], f: T -> U) {
    is_injective_fn(f) implies
    s.subset(set_image(seq_intersection(family), f)) = forall(i: Nat) {
        s.subset(set_image_family(family, f, i))
    }
} by {
    if is_injective_fn(f) {
        set_image_seq_intersection_of_injective(family, f)
        set_subset_seq_intersection_iff(s, set_image_family(family, f))
        s.subset(set_image(seq_intersection(family), f)) = forall(i: Nat) {
            s.subset(set_image_family(family, f, i))
        }
    }
}

/// A set is contained in a bijective image of a sequence intersection exactly when it is contained in every term image.
theorem set_subset_image_seq_intersection_iff_of_bijection[T: Inhabited, U](s: Set[U],
    family: Nat -> Set[T], f: T -> U) {
    is_bijection_fn(f) implies
    s.subset(set_image(seq_intersection(family), f)) = forall(i: Nat) {
        s.subset(set_image_family(family, f, i))
    }
} by {
    if is_bijection_fn(f) {
        set_image_seq_intersection_of_bijection(family, f)
        set_subset_seq_intersection_iff(s, set_image_family(family, f))
        s.subset(set_image(seq_intersection(family), f)) = forall(i: Nat) {
            s.subset(set_image_family(family, f, i))
        }
    }
}

/// The sequence union of constantly empty sets is empty.
theorem set_seq_union_empty_family[K] {
    seq_union(set_empty_family[K, Nat]) = Set[K].empty_set
} by {
    set_sSup_eq_seq_union(set_empty_family[K, Nat])
    set_sSup_empty_family[K, Nat]
    seq_union(set_empty_family[K, Nat]) = Set[K].empty_set
}

/// The sequence intersection of constantly universal sets is universal.
theorem set_seq_intersection_universal_family[K] {
    seq_intersection(set_universal_family[K, Nat]) = Set[K].universal_set
} by {
    set_sInf_eq_seq_intersection(set_universal_family[K, Nat])
    set_sInf_universal_family[K, Nat]
    seq_intersection(set_universal_family[K, Nat]) = Set[K].universal_set
}

/// The sequence union of a constant family is the constant member.
theorem set_seq_union_constant_family[K](s: Set[K]) {
    seq_union(set_constant_family[K, Nat](s)) = s
} by {
    set_sSup_eq_seq_union(set_constant_family[K, Nat](s))
    set_sSup_constant_family[K, Nat](s)
    seq_union(set_constant_family[K, Nat](s)) = s
}

/// The sequence intersection of a constant family is the constant member.
theorem set_seq_intersection_constant_family[K](s: Set[K]) {
    seq_intersection(set_constant_family[K, Nat](s)) = s
} by {
    set_sInf_eq_seq_intersection(set_constant_family[K, Nat](s))
    set_sInf_constant_family[K, Nat](s)
    seq_intersection(set_constant_family[K, Nat](s)) = s
}

/// The sequence union of constantly universal sets is universal.
theorem set_seq_union_universal_family[K] {
    seq_union(set_universal_family[K, Nat]) = Set[K].universal_set
} by {
    set_seq_union_constant_family(Set[K].universal_set)
    forall(i: Nat) {
        set_universal_family[K, Nat](i) = set_constant_family[K, Nat](Set[K].universal_set, i)
    }
    set_sSup_eq_of_family_subset(set_universal_family[K, Nat],
        set_constant_family[K, Nat](Set[K].universal_set))
    set_sSup_eq_seq_union(set_universal_family[K, Nat])
    set_sSup_eq_seq_union(set_constant_family[K, Nat](Set[K].universal_set))
    seq_union(set_universal_family[K, Nat]) = Set[K].universal_set
}

/// The sequence intersection of constantly empty sets is empty.
theorem set_seq_intersection_empty_family[K] {
    seq_intersection(set_empty_family[K, Nat]) = Set[K].empty_set
} by {
    set_seq_intersection_constant_family(Set[K].empty_set)
    forall(i: Nat) {
        set_empty_family[K, Nat](i) = set_constant_family[K, Nat](Set[K].empty_set, i)
    }
    set_sInf_eq_of_family_subset(set_empty_family[K, Nat],
        set_constant_family[K, Nat](Set[K].empty_set))
    set_sInf_eq_seq_intersection(set_empty_family[K, Nat])
    set_sInf_eq_seq_intersection(set_constant_family[K, Nat](Set[K].empty_set))
    seq_intersection(set_empty_family[K, Nat]) = Set[K].empty_set
}

/// The sequence union of complements is the complement of the sequence intersection.
theorem set_seq_union_complement_eq_seq_intersection_complement[K](family: Nat -> Set[K]) {
    seq_union(seq_complement(family)) = seq_intersection(family).c
} by {
    seq_union_complement_eq_intersection_complement(family)
}

/// The sequence intersection of complements is the complement of the sequence union.
theorem set_seq_intersection_complement_eq_seq_union_complement[K](family: Nat -> Set[K]) {
    seq_intersection(seq_complement(family)) = seq_union(family).c
} by {
    seq_intersection_complement_eq_union_complement(family)
}

/// The named complement family agrees with sequence complement.
theorem set_complement_family_eq_seq_complement[K](family: Nat -> Set[K]) {
    set_complement_family(family) = seq_complement(family)
} by {
    let u = set_complement_family(family)
    let v = seq_complement(family)
    forall(i: Nat) {
        u(i) = set_complement_family(family, i)
        set_complement_family(family, i) = family(i).c
        v(i) = seq_complement(family, i)
        seq_complement(family, i) = family(i).c
        u(i) = v(i)
    }
    function_extensionality(u, v)
}

/// Binary infimum distributes over a sequence union on the right.
theorem set_inf_seq_union_distrib_left[K](s: Set[K], family: Nat -> Set[K]) {
    set_inf(s, seq_union(family)) = seq_union(set_inf_family(s, family))
} by {
    set_sSup_eq_seq_union(family)
    set_inf_sSup_distrib_left(s, family)
    set_sSup_eq_seq_union(set_inf_family(s, family))
    set_inf(s, seq_union(family)) = seq_union(set_inf_family(s, family))
}

/// Binary infimum distributes over a sequence union on the left.
theorem set_inf_seq_union_distrib_right[K](family: Nat -> Set[K], s: Set[K]) {
    set_inf(seq_union(family), s) = seq_union(set_inf_family_right(family, s))
} by {
    set_sSup_eq_seq_union(family)
    set_inf_sSup_distrib_right(family, s)
    set_sSup_eq_seq_union(set_inf_family_right(family, s))
    set_inf(seq_union(family), s) = seq_union(set_inf_family_right(family, s))
}

/// Binary supremum distributes over a sequence intersection on the right.
theorem set_sup_seq_intersection_distrib_left[K](s: Set[K], family: Nat -> Set[K]) {
    set_sup(s, seq_intersection(family)) = seq_intersection(set_sup_family(s, family))
} by {
    set_sInf_eq_seq_intersection(family)
    set_sup_sInf_distrib_left(s, family)
    set_sInf_eq_seq_intersection(set_sup_family(s, family))
    set_sup(s, seq_intersection(family)) = seq_intersection(set_sup_family(s, family))
}

/// Binary supremum distributes over a sequence intersection on the left.
theorem set_sup_seq_intersection_distrib_right[K](family: Nat -> Set[K], s: Set[K]) {
    set_sup(seq_intersection(family), s) = seq_intersection(set_sup_family_right(family, s))
} by {
    set_sInf_eq_seq_intersection(family)
    set_sup_sInf_distrib_right(family, s)
    set_sInf_eq_seq_intersection(set_sup_family_right(family, s))
    set_sup(seq_intersection(family), s) = seq_intersection(set_sup_family_right(family, s))
}

/// Binary supremum distributes over a sequence union on the right.
theorem set_sup_seq_union_distrib_left[K](s: Set[K], family: Nat -> Set[K]) {
    set_sup(s, seq_union(family)) = seq_union(set_sup_family(s, family))
} by {
    set_sSup_eq_seq_union(family)
    set_sup_sSup_distrib_left(s, family)
    set_sSup_eq_seq_union(set_sup_family(s, family))
    set_sup(s, seq_union(family)) = seq_union(set_sup_family(s, family))
}

/// Binary supremum distributes over a sequence union on the left.
theorem set_sup_seq_union_distrib_right[K](family: Nat -> Set[K], s: Set[K]) {
    set_sup(seq_union(family), s) = seq_union(set_sup_family_right(family, s))
} by {
    set_sSup_eq_seq_union(family)
    set_sup_sSup_distrib_right(family, s)
    set_sSup_eq_seq_union(set_sup_family_right(family, s))
    set_sup(seq_union(family), s) = seq_union(set_sup_family_right(family, s))
}

/// Binary infimum distributes over a sequence intersection on the right.
theorem set_inf_seq_intersection_distrib_left[K](s: Set[K], family: Nat -> Set[K]) {
    set_inf(s, seq_intersection(family)) = seq_intersection(set_inf_family(s, family))
} by {
    set_sInf_eq_seq_intersection(family)
    set_inf_sInf_distrib_left(s, family)
    set_sInf_eq_seq_intersection(set_inf_family(s, family))
    set_inf(s, seq_intersection(family)) = seq_intersection(set_inf_family(s, family))
}

/// Binary infimum distributes over a sequence intersection on the left.
theorem set_inf_seq_intersection_distrib_right[K](family: Nat -> Set[K], s: Set[K]) {
    set_inf(seq_intersection(family), s) = seq_intersection(set_inf_family_right(family, s))
} by {
    set_sInf_eq_seq_intersection(family)
    set_inf_sInf_distrib_right(family, s)
    set_sInf_eq_seq_intersection(set_inf_family_right(family, s))
    set_inf(seq_intersection(family), s) = seq_intersection(set_inf_family_right(family, s))
}

/// Reindexing a sequence gives a sequence union contained in the original sequence union.
theorem set_seq_union_reindex_subset[K](family: Nat -> Set[K], h: Nat -> Nat) {
    seq_union(set_reindex_family(family, h)).subset(seq_union(family))
} by {
    set_sSup_reindex_subset(family, h)
    set_sSup_eq_seq_union(set_reindex_family(family, h))
    set_sSup_eq_seq_union(family)
    set_sSup(set_reindex_family(family, h)).subset(set_sSup(family))
    seq_union(set_reindex_family(family, h)).subset(seq_union(family))
}

/// The original sequence intersection is contained in a reindexed sequence intersection.
theorem set_seq_intersection_subset_reindex[K](family: Nat -> Set[K], h: Nat -> Nat) {
    seq_intersection(family).subset(seq_intersection(set_reindex_family(family, h)))
} by {
    set_sInf_subset_reindex(family, h)
    set_sInf_eq_seq_intersection(family)
    set_sInf_eq_seq_intersection(set_reindex_family(family, h))
    set_sInf(family).subset(set_sInf(set_reindex_family(family, h)))
    seq_intersection(family).subset(seq_intersection(set_reindex_family(family, h)))
}

/// Reindexing a sequence by a map with a section preserves the sequence union.
theorem set_seq_union_reindex_eq_of_section[K](family: Nat -> Set[K],
    h: Nat -> Nat, g: Nat -> Nat) {
    (forall(i: Nat) { h(g(i)) = i }) implies
    seq_union(set_reindex_family(family, h)) = seq_union(family)
} by {
    if forall(i: Nat) { h(g(i)) = i } {
        set_sSup_reindex_eq_of_section(family, h, g)
        set_sSup_eq_seq_union(set_reindex_family(family, h))
        set_sSup_eq_seq_union(family)
        set_sSup(set_reindex_family(family, h)) = set_sSup(family)
        seq_union(set_reindex_family(family, h)) = seq_union(family)
    }
}

/// Reindexing a sequence by a map with a section preserves the sequence intersection.
theorem set_seq_intersection_reindex_eq_of_section[K](family: Nat -> Set[K],
    h: Nat -> Nat, g: Nat -> Nat) {
    (forall(i: Nat) { h(g(i)) = i }) implies
    seq_intersection(set_reindex_family(family, h)) = seq_intersection(family)
} by {
    if forall(i: Nat) { h(g(i)) = i } {
        set_sInf_reindex_eq_of_section(family, h, g)
        set_sInf_eq_seq_intersection(set_reindex_family(family, h))
        set_sInf_eq_seq_intersection(family)
        set_sInf(set_reindex_family(family, h)) = set_sInf(family)
        seq_intersection(set_reindex_family(family, h)) = seq_intersection(family)
    }
}

/// Reindexing a sequence by a map with a right inverse preserves the sequence union.
theorem set_seq_union_reindex_eq_of_right_inverse[K](family: Nat -> Set[K],
    h: Nat -> Nat, g: Nat -> Nat) {
    is_right_inverse_fn(h, g) implies
    seq_union(set_reindex_family(family, h)) = seq_union(family)
} by {
    if is_right_inverse_fn(h, g) {
        set_sSup_reindex_eq_of_right_inverse(family, h, g)
        set_sSup_eq_seq_union(set_reindex_family(family, h))
        set_sSup_eq_seq_union(family)
        set_sSup(set_reindex_family(family, h)) = set_sSup(family)
        seq_union(set_reindex_family(family, h)) = seq_union(family)
    }
}

/// Reindexing a sequence by a map with a right inverse preserves the sequence intersection.
theorem set_seq_intersection_reindex_eq_of_right_inverse[K](family: Nat -> Set[K],
    h: Nat -> Nat, g: Nat -> Nat) {
    is_right_inverse_fn(h, g) implies
    seq_intersection(set_reindex_family(family, h)) = seq_intersection(family)
} by {
    if is_right_inverse_fn(h, g) {
        set_sInf_reindex_eq_of_right_inverse(family, h, g)
        set_sInf_eq_seq_intersection(set_reindex_family(family, h))
        set_sInf_eq_seq_intersection(family)
        set_sInf(set_reindex_family(family, h)) = set_sInf(family)
        seq_intersection(set_reindex_family(family, h)) = seq_intersection(family)
    }
}

/// Reindexing a sequence by a surjective map preserves the sequence union.
theorem set_seq_union_reindex_eq_of_surjective[K](family: Nat -> Set[K], h: Nat -> Nat) {
    is_surjective_fn(h) implies
    seq_union(set_reindex_family(family, h)) = seq_union(family)
} by {
    if is_surjective_fn(h) {
        set_sSup_reindex_eq_of_surjective(family, h)
        set_sSup_eq_seq_union(set_reindex_family(family, h))
        set_sSup_eq_seq_union(family)
        set_sSup(set_reindex_family(family, h)) = set_sSup(family)
        seq_union(set_reindex_family(family, h)) = seq_union(family)
    }
}

/// Reindexing a sequence by a surjective map preserves the sequence intersection.
theorem set_seq_intersection_reindex_eq_of_surjective[K](family: Nat -> Set[K],
    h: Nat -> Nat) {
    is_surjective_fn(h) implies
    seq_intersection(set_reindex_family(family, h)) = seq_intersection(family)
} by {
    if is_surjective_fn(h) {
        set_sInf_reindex_eq_of_surjective(family, h)
        set_sInf_eq_seq_intersection(set_reindex_family(family, h))
        set_sInf_eq_seq_intersection(family)
        set_sInf(set_reindex_family(family, h)) = set_sInf(family)
        seq_intersection(set_reindex_family(family, h)) = seq_intersection(family)
    }
}

/// Reindexing a sequence by a bijective map preserves the sequence union.
theorem set_seq_union_reindex_eq_of_bijection[K](family: Nat -> Set[K], h: Nat -> Nat) {
    is_bijection_fn(h) implies
    seq_union(set_reindex_family(family, h)) = seq_union(family)
} by {
    if is_bijection_fn(h) {
        set_sSup_reindex_eq_of_bijection(family, h)
        set_sSup_eq_seq_union(set_reindex_family(family, h))
        set_sSup_eq_seq_union(family)
        set_sSup(set_reindex_family(family, h)) = set_sSup(family)
        seq_union(set_reindex_family(family, h)) = seq_union(family)
    }
}

/// Reindexing a sequence by a bijective map preserves the sequence intersection.
theorem set_seq_intersection_reindex_eq_of_bijection[K](family: Nat -> Set[K],
    h: Nat -> Nat) {
    is_bijection_fn(h) implies
    seq_intersection(set_reindex_family(family, h)) = seq_intersection(family)
} by {
    if is_bijection_fn(h) {
        set_sInf_reindex_eq_of_bijection(family, h)
        set_sInf_eq_seq_intersection(set_reindex_family(family, h))
        set_sInf_eq_seq_intersection(family)
        set_sInf(set_reindex_family(family, h)) = set_sInf(family)
        seq_intersection(set_reindex_family(family, h)) = seq_intersection(family)
    }
}

/// The sequence formed by taking sequence unions over the right index.
define set_seq_union_right_family[K](family: (Nat, Nat) -> Set[K], i: Nat) -> Set[K] {
    seq_union(set_right_slice_family(family, i))
}

/// The sequence formed by taking sequence unions over the left index.
define set_seq_union_left_family[K](family: (Nat, Nat) -> Set[K], j: Nat) -> Set[K] {
    seq_union(set_left_slice_family(family, j))
}

/// The sequence formed by taking sequence intersections over the right index.
define set_seq_intersection_right_family[K](family: (Nat, Nat) -> Set[K],
    i: Nat) -> Set[K] {
    seq_intersection(set_right_slice_family(family, i))
}

/// The sequence formed by taking sequence intersections over the left index.
define set_seq_intersection_left_family[K](family: (Nat, Nat) -> Set[K],
    j: Nat) -> Set[K] {
    seq_intersection(set_left_slice_family(family, j))
}

/// Right slices of a complement binary family are complement families of right slices.
theorem set_right_slice_complement_binary_family_eq[K](
    family: (Nat, Nat) -> Set[K], i: Nat
) {
    set_right_slice_family(set_complement_binary_family(family), i) =
        set_complement_family(set_right_slice_family(family, i))
} by {
    let u = set_right_slice_family(set_complement_binary_family(family), i)
    let v = set_complement_family(set_right_slice_family(family, i))
    forall(j: Nat) {
        u(j) = set_right_slice_family(set_complement_binary_family(family), i, j)
        set_right_slice_family(set_complement_binary_family(family), i, j) =
            set_complement_binary_family(family, i, j)
        set_complement_binary_family(family, i, j) = family(i, j).c
        v(j) = set_complement_family(set_right_slice_family(family, i), j)
        set_complement_family(set_right_slice_family(family, i), j) =
            set_right_slice_family(family, i, j).c
        set_right_slice_family(family, i, j) = family(i, j)
        v(j) = family(i, j).c
        u(j) = v(j)
    }
    function_extensionality(u, v)
}

/// Left slices of a complement binary family are complement families of left slices.
theorem set_left_slice_complement_binary_family_eq[K](
    family: (Nat, Nat) -> Set[K], j: Nat
) {
    set_left_slice_family(set_complement_binary_family(family), j) =
        set_complement_family(set_left_slice_family(family, j))
} by {
    let u = set_left_slice_family(set_complement_binary_family(family), j)
    let v = set_complement_family(set_left_slice_family(family, j))
    forall(i: Nat) {
        u(i) = set_left_slice_family(set_complement_binary_family(family), j, i)
        set_left_slice_family(set_complement_binary_family(family), j, i) =
            set_complement_binary_family(family, i, j)
        set_complement_binary_family(family, i, j) = family(i, j).c
        v(i) = set_complement_family(set_left_slice_family(family, j), i)
        set_complement_family(set_left_slice_family(family, j), i) =
            set_left_slice_family(family, j, i).c
        set_left_slice_family(family, j, i) = family(i, j)
        v(i) = family(i, j).c
        u(i) = v(i)
    }
    function_extensionality(u, v)
}

/// Right-index sequence unions of complement members are complements of right-index sequence intersections.
theorem set_seq_union_right_complement_binary_family_eq[K](
    family: (Nat, Nat) -> Set[K]
) {
    set_seq_union_right_family(set_complement_binary_family(family)) =
        seq_complement(set_seq_intersection_right_family(family))
} by {
    let u = set_seq_union_right_family(set_complement_binary_family(family))
    let v = seq_complement(set_seq_intersection_right_family(family))
    forall(i: Nat) {
        u(i) = seq_union(set_right_slice_family(set_complement_binary_family(family), i))
        set_right_slice_complement_binary_family_eq(family, i)
        set_complement_family_eq_seq_complement(set_right_slice_family(family, i))
        set_right_slice_family(set_complement_binary_family(family), i) =
            seq_complement(set_right_slice_family(family, i))
        seq_union_complement_eq_intersection_complement(set_right_slice_family(family, i))
        u(i) = seq_intersection(set_right_slice_family(family, i)).c
        set_seq_intersection_right_family(family, i) =
            seq_intersection(set_right_slice_family(family, i))
        v(i) = seq_complement(set_seq_intersection_right_family(family), i)
        seq_complement(set_seq_intersection_right_family(family), i) =
            set_seq_intersection_right_family(family, i).c
        v(i) = seq_intersection(set_right_slice_family(family, i)).c
        u(i) = v(i)
    }
    function_extensionality(u, v)
}

/// Left-index sequence unions of complement members are complements of left-index sequence intersections.
theorem set_seq_union_left_complement_binary_family_eq[K](
    family: (Nat, Nat) -> Set[K]
) {
    set_seq_union_left_family(set_complement_binary_family(family)) =
        seq_complement(set_seq_intersection_left_family(family))
} by {
    let u = set_seq_union_left_family(set_complement_binary_family(family))
    let v = seq_complement(set_seq_intersection_left_family(family))
    forall(j: Nat) {
        u(j) = seq_union(set_left_slice_family(set_complement_binary_family(family), j))
        set_left_slice_complement_binary_family_eq(family, j)
        set_complement_family_eq_seq_complement(set_left_slice_family(family, j))
        set_left_slice_family(set_complement_binary_family(family), j) =
            seq_complement(set_left_slice_family(family, j))
        seq_union_complement_eq_intersection_complement(set_left_slice_family(family, j))
        u(j) = seq_intersection(set_left_slice_family(family, j)).c
        set_seq_intersection_left_family(family, j) =
            seq_intersection(set_left_slice_family(family, j))
        v(j) = seq_complement(set_seq_intersection_left_family(family), j)
        seq_complement(set_seq_intersection_left_family(family), j) =
            set_seq_intersection_left_family(family, j).c
        v(j) = seq_intersection(set_left_slice_family(family, j)).c
        u(j) = v(j)
    }
    function_extensionality(u, v)
}

/// Right-index sequence intersections of complement members are complements of right-index sequence unions.
theorem set_seq_intersection_right_complement_binary_family_eq[K](
    family: (Nat, Nat) -> Set[K]
) {
    set_seq_intersection_right_family(set_complement_binary_family(family)) =
        seq_complement(set_seq_union_right_family(family))
} by {
    let u = set_seq_intersection_right_family(set_complement_binary_family(family))
    let v = seq_complement(set_seq_union_right_family(family))
    forall(i: Nat) {
        u(i) = seq_intersection(set_right_slice_family(set_complement_binary_family(family), i))
        set_right_slice_complement_binary_family_eq(family, i)
        set_complement_family_eq_seq_complement(set_right_slice_family(family, i))
        set_right_slice_family(set_complement_binary_family(family), i) =
            seq_complement(set_right_slice_family(family, i))
        seq_intersection_complement_eq_union_complement(set_right_slice_family(family, i))
        u(i) = seq_union(set_right_slice_family(family, i)).c
        set_seq_union_right_family(family, i) = seq_union(set_right_slice_family(family, i))
        v(i) = seq_complement(set_seq_union_right_family(family), i)
        seq_complement(set_seq_union_right_family(family), i) =
            set_seq_union_right_family(family, i).c
        v(i) = seq_union(set_right_slice_family(family, i)).c
        u(i) = v(i)
    }
    function_extensionality(u, v)
}

/// Left-index sequence intersections of complement members are complements of left-index sequence unions.
theorem set_seq_intersection_left_complement_binary_family_eq[K](
    family: (Nat, Nat) -> Set[K]
) {
    set_seq_intersection_left_family(set_complement_binary_family(family)) =
        seq_complement(set_seq_union_left_family(family))
} by {
    let u = set_seq_intersection_left_family(set_complement_binary_family(family))
    let v = seq_complement(set_seq_union_left_family(family))
    forall(j: Nat) {
        u(j) = seq_intersection(set_left_slice_family(set_complement_binary_family(family), j))
        set_left_slice_complement_binary_family_eq(family, j)
        set_complement_family_eq_seq_complement(set_left_slice_family(family, j))
        set_left_slice_family(set_complement_binary_family(family), j) =
            seq_complement(set_left_slice_family(family, j))
        seq_intersection_complement_eq_union_complement(set_left_slice_family(family, j))
        u(j) = seq_union(set_left_slice_family(family, j)).c
        set_seq_union_left_family(family, j) = seq_union(set_left_slice_family(family, j))
        v(j) = seq_complement(set_seq_union_left_family(family), j)
        seq_complement(set_seq_union_left_family(family), j) =
            set_seq_union_left_family(family, j).c
        v(j) = seq_union(set_left_slice_family(family, j)).c
        u(j) = v(j)
    }
    function_extensionality(u, v)
}

/// The right-associated iterated union of complement members is the complement of the right-associated iterated intersection.
theorem set_seq_union_right_complement_binary_family_eq_intersection_complement[K](
    family: (Nat, Nat) -> Set[K]
) {
    seq_union(set_seq_union_right_family(set_complement_binary_family(family))) =
        seq_intersection(set_seq_intersection_right_family(family)).c
} by {
    set_seq_union_right_complement_binary_family_eq(family)
    seq_union_complement_eq_intersection_complement(set_seq_intersection_right_family(family))
    seq_union(set_seq_union_right_family(set_complement_binary_family(family))) =
        seq_union(seq_complement(set_seq_intersection_right_family(family)))
    seq_union(set_seq_union_right_family(set_complement_binary_family(family))) =
        seq_intersection(set_seq_intersection_right_family(family)).c
}

/// The left-associated iterated union of complement members is the complement of the left-associated iterated intersection.
theorem set_seq_union_left_complement_binary_family_eq_intersection_complement[K](
    family: (Nat, Nat) -> Set[K]
) {
    seq_union(set_seq_union_left_family(set_complement_binary_family(family))) =
        seq_intersection(set_seq_intersection_left_family(family)).c
} by {
    set_seq_union_left_complement_binary_family_eq(family)
    seq_union_complement_eq_intersection_complement(set_seq_intersection_left_family(family))
    seq_union(set_seq_union_left_family(set_complement_binary_family(family))) =
        seq_union(seq_complement(set_seq_intersection_left_family(family)))
    seq_union(set_seq_union_left_family(set_complement_binary_family(family))) =
        seq_intersection(set_seq_intersection_left_family(family)).c
}

/// The right-associated iterated intersection of complement members is the complement of the right-associated iterated union.
theorem set_seq_intersection_right_complement_binary_family_eq_union_complement[K](
    family: (Nat, Nat) -> Set[K]
) {
    seq_intersection(set_seq_intersection_right_family(set_complement_binary_family(family))) =
        seq_union(set_seq_union_right_family(family)).c
} by {
    set_seq_intersection_right_complement_binary_family_eq(family)
    seq_intersection_complement_eq_union_complement(set_seq_union_right_family(family))
    seq_intersection(set_seq_intersection_right_family(set_complement_binary_family(family))) =
        seq_intersection(seq_complement(set_seq_union_right_family(family)))
    seq_intersection(set_seq_intersection_right_family(set_complement_binary_family(family))) =
        seq_union(set_seq_union_right_family(family)).c
}

/// The left-associated iterated intersection of complement members is the complement of the left-associated iterated union.
theorem set_seq_intersection_left_complement_binary_family_eq_union_complement[K](
    family: (Nat, Nat) -> Set[K]
) {
    seq_intersection(set_seq_intersection_left_family(set_complement_binary_family(family))) =
        seq_union(set_seq_union_left_family(family)).c
} by {
    set_seq_intersection_left_complement_binary_family_eq(family)
    seq_intersection_complement_eq_union_complement(set_seq_union_left_family(family))
    seq_intersection(set_seq_intersection_left_family(set_complement_binary_family(family))) =
        seq_intersection(seq_complement(set_seq_union_left_family(family)))
    seq_intersection(set_seq_intersection_left_family(set_complement_binary_family(family))) =
        seq_union(set_seq_union_left_family(family)).c
}

/// Right-index sequence unions agree with generic right-index suprema.
theorem set_seq_union_right_family_eq_sSup_right[K](family: (Nat, Nat) -> Set[K]) {
    set_seq_union_right_family(family) = set_sSup_right_family(family)
} by {
    let u = set_seq_union_right_family(family)
    let v = set_sSup_right_family(family)
    forall(i: Nat) {
        u(i) = seq_union(set_right_slice_family(family, i))
        set_sSup_eq_seq_union(set_right_slice_family(family, i))
        v(i) = set_sSup(set_right_slice_family(family, i))
        u(i) = v(i)
    }
    function_extensionality(u, v)
}

/// Left-index sequence unions agree with generic left-index suprema.
theorem set_seq_union_left_family_eq_sSup_left[K](family: (Nat, Nat) -> Set[K]) {
    set_seq_union_left_family(family) = set_sSup_left_family(family)
} by {
    let u = set_seq_union_left_family(family)
    let v = set_sSup_left_family(family)
    forall(j: Nat) {
        u(j) = seq_union(set_left_slice_family(family, j))
        set_sSup_eq_seq_union(set_left_slice_family(family, j))
        v(j) = set_sSup(set_left_slice_family(family, j))
        u(j) = v(j)
    }
    function_extensionality(u, v)
}

/// Right-index sequence intersections agree with generic right-index infima.
theorem set_seq_intersection_right_family_eq_sInf_right[K](
    family: (Nat, Nat) -> Set[K]
) {
    set_seq_intersection_right_family(family) = set_sInf_right_family(family)
} by {
    let u = set_seq_intersection_right_family(family)
    let v = set_sInf_right_family(family)
    forall(i: Nat) {
        u(i) = seq_intersection(set_right_slice_family(family, i))
        set_sInf_eq_seq_intersection(set_right_slice_family(family, i))
        v(i) = set_sInf(set_right_slice_family(family, i))
        u(i) = v(i)
    }
    function_extensionality(u, v)
}

/// Left-index sequence intersections agree with generic left-index infima.
theorem set_seq_intersection_left_family_eq_sInf_left[K](
    family: (Nat, Nat) -> Set[K]
) {
    set_seq_intersection_left_family(family) = set_sInf_left_family(family)
} by {
    let u = set_seq_intersection_left_family(family)
    let v = set_sInf_left_family(family)
    forall(j: Nat) {
        u(j) = seq_intersection(set_left_slice_family(family, j))
        set_sInf_eq_seq_intersection(set_left_slice_family(family, j))
        v(j) = set_sInf(set_left_slice_family(family, j))
        u(j) = v(j)
    }
    function_extensionality(u, v)
}

/// Flattening a natural binary family agrees with first taking right-index sequence unions.
theorem set_sSup_pair_family_eq_seq_union_right[K](family: (Nat, Nat) -> Set[K]) {
    set_sSup(set_pair_family(family)) = seq_union(set_seq_union_right_family(family))
} by {
    set_sSup_pair_family_eq_sSup_right(family)
    set_seq_union_right_family_eq_sSup_right(family)
    set_sSup(set_pair_family(family)) = set_sSup(set_seq_union_right_family(family))
    set_sSup_eq_seq_union(set_seq_union_right_family(family))
    set_sSup(set_pair_family(family)) = seq_union(set_seq_union_right_family(family))
}

/// Flattening a natural binary family agrees with first taking left-index sequence unions.
theorem set_sSup_pair_family_eq_seq_union_left[K](family: (Nat, Nat) -> Set[K]) {
    set_sSup(set_pair_family(family)) = seq_union(set_seq_union_left_family(family))
} by {
    set_sSup_pair_family_eq_sSup_left(family)
    set_seq_union_left_family_eq_sSup_left(family)
    set_sSup(set_pair_family(family)) = set_sSup(set_seq_union_left_family(family))
    set_sSup_eq_seq_union(set_seq_union_left_family(family))
    set_sSup(set_pair_family(family)) = seq_union(set_seq_union_left_family(family))
}

/// Flattening a natural binary family agrees with first taking right-index sequence intersections.
theorem set_sInf_pair_family_eq_seq_intersection_right[K](
    family: (Nat, Nat) -> Set[K]
) {
    set_sInf(set_pair_family(family)) =
        seq_intersection(set_seq_intersection_right_family(family))
} by {
    set_sInf_pair_family_eq_sInf_right(family)
    set_seq_intersection_right_family_eq_sInf_right(family)
    set_sInf(set_pair_family(family)) =
        set_sInf(set_seq_intersection_right_family(family))
    set_sInf_eq_seq_intersection(set_seq_intersection_right_family(family))
    set_sInf(set_pair_family(family)) =
        seq_intersection(set_seq_intersection_right_family(family))
}

/// Flattening a natural binary family agrees with first taking left-index sequence intersections.
theorem set_sInf_pair_family_eq_seq_intersection_left[K](
    family: (Nat, Nat) -> Set[K]
) {
    set_sInf(set_pair_family(family)) =
        seq_intersection(set_seq_intersection_left_family(family))
} by {
    set_sInf_pair_family_eq_sInf_left(family)
    set_seq_intersection_left_family_eq_sInf_left(family)
    set_sInf(set_pair_family(family)) =
        set_sInf(set_seq_intersection_left_family(family))
    set_sInf_eq_seq_intersection(set_seq_intersection_left_family(family))
    set_sInf(set_pair_family(family)) =
        seq_intersection(set_seq_intersection_left_family(family))
}

/// The two orders of iterated sequence union over a natural binary family agree.
theorem set_seq_union_right_family_eq_seq_union_left[K](family: (Nat, Nat) -> Set[K]) {
    seq_union(set_seq_union_right_family(family)) =
        seq_union(set_seq_union_left_family(family))
} by {
    set_sSup_pair_family_eq_seq_union_right(family)
    set_sSup_pair_family_eq_seq_union_left(family)
}

/// The two orders of iterated sequence intersection over a natural binary family agree.
theorem set_seq_intersection_right_family_eq_seq_intersection_left[K](
    family: (Nat, Nat) -> Set[K]
) {
    seq_intersection(set_seq_intersection_right_family(family)) =
        seq_intersection(set_seq_intersection_left_family(family))
} by {
    set_sInf_pair_family_eq_seq_intersection_right(family)
    set_sInf_pair_family_eq_seq_intersection_left(family)
}

/// Membership in an iterated sequence union is membership in some indexed member.
theorem set_seq_union_right_family_contains_eq[K](family: (Nat, Nat) -> Set[K],
    x: K) {
    seq_union(set_seq_union_right_family(family)).contains(x) = exists(i: Nat) {
        exists(j: Nat) {
            family(i, j).contains(x)
        }
    }
} by {
    set_sSup_pair_family_eq_seq_union_right(family)
    set_sSup_pair_family_contains_eq(family, x)
    seq_union(set_seq_union_right_family(family)).contains(x) = exists(i: Nat) {
        exists(j: Nat) {
            family(i, j).contains(x)
        }
    }
}

/// Membership in the other iterated sequence union is membership in some indexed member.
theorem set_seq_union_left_family_contains_eq[K](family: (Nat, Nat) -> Set[K],
    x: K) {
    seq_union(set_seq_union_left_family(family)).contains(x) = exists(i: Nat) {
        exists(j: Nat) {
            family(i, j).contains(x)
        }
    }
} by {
    set_sSup_pair_family_eq_seq_union_left(family)
    set_sSup_pair_family_contains_eq(family, x)
    seq_union(set_seq_union_left_family(family)).contains(x) = exists(i: Nat) {
        exists(j: Nat) {
            family(i, j).contains(x)
        }
    }
}

/// Membership in an iterated sequence intersection is membership in every indexed member.
theorem set_seq_intersection_right_family_contains_eq[K](family: (Nat, Nat) -> Set[K],
    x: K) {
    seq_intersection(set_seq_intersection_right_family(family)).contains(x) =
    forall(i: Nat) {
        forall(j: Nat) {
            family(i, j).contains(x)
        }
    }
} by {
    set_sInf_pair_family_eq_seq_intersection_right(family)
    set_sInf_pair_family_contains_eq(family, x)
    seq_intersection(set_seq_intersection_right_family(family)).contains(x) =
    forall(i: Nat) {
        forall(j: Nat) {
            family(i, j).contains(x)
        }
    }
}

/// Membership in the other iterated sequence intersection is membership in every indexed member.
theorem set_seq_intersection_left_family_contains_eq[K](family: (Nat, Nat) -> Set[K],
    x: K) {
    seq_intersection(set_seq_intersection_left_family(family)).contains(x) =
    forall(i: Nat) {
        forall(j: Nat) {
            family(i, j).contains(x)
        }
    }
} by {
    set_sInf_pair_family_eq_seq_intersection_left(family)
    set_sInf_pair_family_contains_eq(family, x)
    seq_intersection(set_seq_intersection_left_family(family)).contains(x) =
    forall(i: Nat) {
        forall(j: Nat) {
            family(i, j).contains(x)
        }
    }
}

/// An iterated sequence union is contained in a set exactly when every indexed member is.
theorem set_seq_union_right_family_subset_iff[K](family: (Nat, Nat) -> Set[K],
    s: Set[K]) {
    seq_union(set_seq_union_right_family(family)).subset(s) = forall(i: Nat) {
        forall(j: Nat) {
            family(i, j).subset(s)
        }
    }
} by {
    set_sSup_pair_family_eq_seq_union_right(family)
    set_sSup_pair_family_subset_iff(family, s)
    seq_union(set_seq_union_right_family(family)).subset(s) = forall(i: Nat) {
        forall(j: Nat) {
            family(i, j).subset(s)
        }
    }
}

/// The other iterated sequence union is contained in a set exactly when every indexed member is.
theorem set_seq_union_left_family_subset_iff[K](family: (Nat, Nat) -> Set[K],
    s: Set[K]) {
    seq_union(set_seq_union_left_family(family)).subset(s) = forall(i: Nat) {
        forall(j: Nat) {
            family(i, j).subset(s)
        }
    }
} by {
    set_sSup_pair_family_eq_seq_union_left(family)
    set_sSup_pair_family_subset_iff(family, s)
    seq_union(set_seq_union_left_family(family)).subset(s) = forall(i: Nat) {
        forall(j: Nat) {
            family(i, j).subset(s)
        }
    }
}

/// A set is contained in an iterated sequence intersection exactly when it is contained in every indexed member.
theorem set_subset_seq_intersection_right_family_iff[K](s: Set[K],
    family: (Nat, Nat) -> Set[K]) {
    s.subset(seq_intersection(set_seq_intersection_right_family(family))) =
    forall(i: Nat) {
        forall(j: Nat) {
            s.subset(family(i, j))
        }
    }
} by {
    set_sInf_pair_family_eq_seq_intersection_right(family)
    set_subset_sInf_pair_family_iff(s, family)
    s.subset(seq_intersection(set_seq_intersection_right_family(family))) =
    forall(i: Nat) {
        forall(j: Nat) {
            s.subset(family(i, j))
        }
    }
}

/// A set is contained in the other iterated sequence intersection exactly when it is contained in every indexed member.
theorem set_subset_seq_intersection_left_family_iff[K](s: Set[K],
    family: (Nat, Nat) -> Set[K]) {
    s.subset(seq_intersection(set_seq_intersection_left_family(family))) =
    forall(i: Nat) {
        forall(j: Nat) {
            s.subset(family(i, j))
        }
    }
} by {
    set_sInf_pair_family_eq_seq_intersection_left(family)
    set_subset_sInf_pair_family_iff(s, family)
    s.subset(seq_intersection(set_seq_intersection_left_family(family))) =
    forall(i: Nat) {
        forall(j: Nat) {
            s.subset(family(i, j))
        }
    }
}

/// Preimage commutes with an iterated sequence union over a natural binary family.
theorem set_preimage_seq_union_right_family[T, U](f: T -> U,
    family: (Nat, Nat) -> Set[U]) {
    set_preimage(f, seq_union(set_seq_union_right_family(family))) =
        seq_union(set_seq_union_right_family(set_preimage_binary_family(f, family)))
} by {
    set_sSup_pair_family_eq_seq_union_right(family)
    set_sSup_pair_family_eq_seq_union_right(set_preimage_binary_family(f, family))
    set_preimage_sSup_pair_family(f, family)
    set_preimage(f, set_sSup(set_pair_family(family))) =
        set_sSup(set_pair_family(set_preimage_binary_family(f, family)))
    set_preimage(f, seq_union(set_seq_union_right_family(family))) =
        seq_union(set_seq_union_right_family(set_preimage_binary_family(f, family)))
}

/// Preimage commutes with the other iterated sequence union over a natural binary family.
theorem set_preimage_seq_union_left_family[T, U](f: T -> U,
    family: (Nat, Nat) -> Set[U]) {
    set_preimage(f, seq_union(set_seq_union_left_family(family))) =
        seq_union(set_seq_union_left_family(set_preimage_binary_family(f, family)))
} by {
    set_sSup_pair_family_eq_seq_union_left(family)
    set_sSup_pair_family_eq_seq_union_left(set_preimage_binary_family(f, family))
    set_preimage_sSup_pair_family(f, family)
    set_preimage(f, set_sSup(set_pair_family(family))) =
        set_sSup(set_pair_family(set_preimage_binary_family(f, family)))
    set_preimage(f, seq_union(set_seq_union_left_family(family))) =
        seq_union(set_seq_union_left_family(set_preimage_binary_family(f, family)))
}

/// Preimage commutes with an iterated sequence intersection over a natural binary family.
theorem set_preimage_seq_intersection_right_family[T, U](f: T -> U,
    family: (Nat, Nat) -> Set[U]) {
    set_preimage(f, seq_intersection(set_seq_intersection_right_family(family))) =
        seq_intersection(
            set_seq_intersection_right_family(set_preimage_binary_family(f, family)))
} by {
    set_sInf_pair_family_eq_seq_intersection_right(family)
    set_sInf_pair_family_eq_seq_intersection_right(set_preimage_binary_family(f, family))
    set_preimage_sInf_pair_family(f, family)
    set_preimage(f, set_sInf(set_pair_family(family))) =
        set_sInf(set_pair_family(set_preimage_binary_family(f, family)))
    set_preimage(f, seq_intersection(set_seq_intersection_right_family(family))) =
        seq_intersection(
            set_seq_intersection_right_family(set_preimage_binary_family(f, family)))
}

/// Preimage commutes with the other iterated sequence intersection over a natural binary family.
theorem set_preimage_seq_intersection_left_family[T, U](f: T -> U,
    family: (Nat, Nat) -> Set[U]) {
    set_preimage(f, seq_intersection(set_seq_intersection_left_family(family))) =
        seq_intersection(
            set_seq_intersection_left_family(set_preimage_binary_family(f, family)))
} by {
    set_sInf_pair_family_eq_seq_intersection_left(family)
    set_sInf_pair_family_eq_seq_intersection_left(set_preimage_binary_family(f, family))
    set_preimage_sInf_pair_family(f, family)
    set_preimage(f, set_sInf(set_pair_family(family))) =
        set_sInf(set_pair_family(set_preimage_binary_family(f, family)))
    set_preimage(f, seq_intersection(set_seq_intersection_left_family(family))) =
        seq_intersection(
            set_seq_intersection_left_family(set_preimage_binary_family(f, family)))
}

/// Image commutes with an iterated sequence union over a natural binary family.
theorem set_image_seq_union_right_family[T, U](family: (Nat, Nat) -> Set[T],
    f: T -> U) {
    set_image(seq_union(set_seq_union_right_family(family)), f) =
        seq_union(set_seq_union_right_family(set_image_binary_family(family, f)))
} by {
    set_sSup_pair_family_eq_seq_union_right(family)
    set_sSup_pair_family_eq_seq_union_right(set_image_binary_family(family, f))
    set_image_sSup_pair_family(family, f)
    set_image(set_sSup(set_pair_family(family)), f) =
        set_sSup(set_pair_family(set_image_binary_family(family, f)))
    set_image(seq_union(set_seq_union_right_family(family)), f) =
        seq_union(set_seq_union_right_family(set_image_binary_family(family, f)))
}

/// Image commutes with the other iterated sequence union over a natural binary family.
theorem set_image_seq_union_left_family[T, U](family: (Nat, Nat) -> Set[T],
    f: T -> U) {
    set_image(seq_union(set_seq_union_left_family(family)), f) =
        seq_union(set_seq_union_left_family(set_image_binary_family(family, f)))
} by {
    set_sSup_pair_family_eq_seq_union_left(family)
    set_sSup_pair_family_eq_seq_union_left(set_image_binary_family(family, f))
    set_image_sSup_pair_family(family, f)
    set_image(set_sSup(set_pair_family(family)), f) =
        set_sSup(set_pair_family(set_image_binary_family(family, f)))
    set_image(seq_union(set_seq_union_left_family(family)), f) =
        seq_union(set_seq_union_left_family(set_image_binary_family(family, f)))
}

/// The image of an iterated sequence intersection is contained in the iterated sequence intersection of the images.
theorem set_image_seq_intersection_right_family_subset[T, U](
    family: (Nat, Nat) -> Set[T], f: T -> U
) {
    set_image(seq_intersection(set_seq_intersection_right_family(family)), f).subset(
        seq_intersection(set_seq_intersection_right_family(set_image_binary_family(family, f))))
} by {
    set_sInf_pair_family_eq_seq_intersection_right(family)
    set_sInf_pair_family_eq_seq_intersection_right(set_image_binary_family(family, f))
    set_image_sInf_pair_family_subset(family, f)
    set_image(set_sInf(set_pair_family(family)), f).subset(
        set_sInf(set_pair_family(set_image_binary_family(family, f))))
    set_image(seq_intersection(set_seq_intersection_right_family(family)), f).subset(
        seq_intersection(set_seq_intersection_right_family(set_image_binary_family(family, f))))
}

/// The image of the other iterated sequence intersection is contained in the iterated sequence intersection of the images.
theorem set_image_seq_intersection_left_family_subset[T, U](
    family: (Nat, Nat) -> Set[T], f: T -> U
) {
    set_image(seq_intersection(set_seq_intersection_left_family(family)), f).subset(
        seq_intersection(set_seq_intersection_left_family(set_image_binary_family(family, f))))
} by {
    set_sInf_pair_family_eq_seq_intersection_left(family)
    set_sInf_pair_family_eq_seq_intersection_left(set_image_binary_family(family, f))
    set_image_sInf_pair_family_subset(family, f)
    set_image(set_sInf(set_pair_family(family)), f).subset(
        set_sInf(set_pair_family(set_image_binary_family(family, f))))
    set_image(seq_intersection(set_seq_intersection_left_family(family)), f).subset(
        seq_intersection(set_seq_intersection_left_family(set_image_binary_family(family, f))))
}

/// Under an injective map, images preserve an iterated sequence intersection.
theorem set_image_seq_intersection_right_family_of_injective[T, U](
    family: (Nat, Nat) -> Set[T], f: T -> U
) {
    is_injective_fn(f) implies
    set_image(seq_intersection(set_seq_intersection_right_family(family)), f) =
        seq_intersection(set_seq_intersection_right_family(set_image_binary_family(family, f)))
} by {
    if is_injective_fn(f) {
        set_sInf_pair_family_eq_seq_intersection_right(family)
        set_sInf_pair_family_eq_seq_intersection_right(set_image_binary_family(family, f))
        set_image_sInf_pair_family_of_injective(family, f, Nat.0, Nat.0)
        set_image(set_sInf(set_pair_family(family)), f) =
            set_sInf(set_pair_family(set_image_binary_family(family, f)))
        set_image(seq_intersection(set_seq_intersection_right_family(family)), f) =
            seq_intersection(
                set_seq_intersection_right_family(set_image_binary_family(family, f)))
    }
}

/// Under an injective map, images preserve the other iterated sequence intersection.
theorem set_image_seq_intersection_left_family_of_injective[T, U](
    family: (Nat, Nat) -> Set[T], f: T -> U
) {
    is_injective_fn(f) implies
    set_image(seq_intersection(set_seq_intersection_left_family(family)), f) =
        seq_intersection(set_seq_intersection_left_family(set_image_binary_family(family, f)))
} by {
    if is_injective_fn(f) {
        set_sInf_pair_family_eq_seq_intersection_left(family)
        set_sInf_pair_family_eq_seq_intersection_left(set_image_binary_family(family, f))
        set_image_sInf_pair_family_of_injective(family, f, Nat.0, Nat.0)
        set_image(set_sInf(set_pair_family(family)), f) =
            set_sInf(set_pair_family(set_image_binary_family(family, f)))
        set_image(seq_intersection(set_seq_intersection_left_family(family)), f) =
            seq_intersection(
                set_seq_intersection_left_family(set_image_binary_family(family, f)))
    }
}

/// The iterated sequence union of constantly empty binary-family members is empty.
theorem set_seq_union_right_empty_binary_family[K] {
    seq_union(set_seq_union_right_family(set_empty_binary_family[K, Nat, Nat])) =
        Set[K].empty_set
} by {
    set_sSup_empty_binary_family[K, Nat, Nat]
    set_sSup_pair_family_eq_seq_union_right(set_empty_binary_family[K, Nat, Nat])
    seq_union(set_seq_union_right_family(set_empty_binary_family[K, Nat, Nat])) =
        Set[K].empty_set
}

/// The other iterated sequence union of constantly empty binary-family members is empty.
theorem set_seq_union_left_empty_binary_family[K] {
    seq_union(set_seq_union_left_family(set_empty_binary_family[K, Nat, Nat])) =
        Set[K].empty_set
} by {
    set_sSup_empty_binary_family[K, Nat, Nat]
    set_sSup_pair_family_eq_seq_union_left(set_empty_binary_family[K, Nat, Nat])
    seq_union(set_seq_union_left_family(set_empty_binary_family[K, Nat, Nat])) =
        Set[K].empty_set
}

/// The iterated sequence intersection of constantly universal binary-family members is universal.
theorem set_seq_intersection_right_universal_binary_family[K] {
    seq_intersection(set_seq_intersection_right_family(set_universal_binary_family[K, Nat, Nat])) =
        Set[K].universal_set
} by {
    set_sInf_universal_binary_family[K, Nat, Nat]
    set_sInf_pair_family_eq_seq_intersection_right(set_universal_binary_family[K, Nat, Nat])
    seq_intersection(set_seq_intersection_right_family(set_universal_binary_family[K, Nat, Nat])) =
        Set[K].universal_set
}

/// The other iterated sequence intersection of constantly universal binary-family members is universal.
theorem set_seq_intersection_left_universal_binary_family[K] {
    seq_intersection(set_seq_intersection_left_family(set_universal_binary_family[K, Nat, Nat])) =
        Set[K].universal_set
} by {
    set_sInf_universal_binary_family[K, Nat, Nat]
    set_sInf_pair_family_eq_seq_intersection_left(set_universal_binary_family[K, Nat, Nat])
    seq_intersection(set_seq_intersection_left_family(set_universal_binary_family[K, Nat, Nat])) =
        Set[K].universal_set
}

/// The iterated sequence union of a constant binary family is the constant member.
theorem set_seq_union_right_constant_binary_family[K](s: Set[K]) {
    seq_union(set_seq_union_right_family(set_constant_binary_family[K, Nat, Nat](s))) = s
} by {
    set_sSup_constant_binary_family(s, Nat.0, Nat.0)
    set_sSup_pair_family_eq_seq_union_right(set_constant_binary_family[K, Nat, Nat](s))
    seq_union(set_seq_union_right_family(set_constant_binary_family[K, Nat, Nat](s))) = s
}

/// The other iterated sequence union of a constant binary family is the constant member.
theorem set_seq_union_left_constant_binary_family[K](s: Set[K]) {
    seq_union(set_seq_union_left_family(set_constant_binary_family[K, Nat, Nat](s))) = s
} by {
    set_sSup_constant_binary_family(s, Nat.0, Nat.0)
    set_sSup_pair_family_eq_seq_union_left(set_constant_binary_family[K, Nat, Nat](s))
    seq_union(set_seq_union_left_family(set_constant_binary_family[K, Nat, Nat](s))) = s
}

/// The iterated sequence intersection of a constant binary family is the constant member.
theorem set_seq_intersection_right_constant_binary_family[K](s: Set[K]) {
    seq_intersection(set_seq_intersection_right_family(set_constant_binary_family[K, Nat, Nat](s))) =
        s
} by {
    set_sInf_constant_binary_family(s, Nat.0, Nat.0)
    set_sInf_pair_family_eq_seq_intersection_right(set_constant_binary_family[K, Nat, Nat](s))
    seq_intersection(set_seq_intersection_right_family(set_constant_binary_family[K, Nat, Nat](s))) =
        s
}

/// The other iterated sequence intersection of a constant binary family is the constant member.
theorem set_seq_intersection_left_constant_binary_family[K](s: Set[K]) {
    seq_intersection(set_seq_intersection_left_family(set_constant_binary_family[K, Nat, Nat](s))) =
        s
} by {
    set_sInf_constant_binary_family(s, Nat.0, Nat.0)
    set_sInf_pair_family_eq_seq_intersection_left(set_constant_binary_family[K, Nat, Nat](s))
    seq_intersection(set_seq_intersection_left_family(set_constant_binary_family[K, Nat, Nat](s))) =
        s
}

/// Pointwise inclusion of natural binary families preserves iterated sequence unions.
theorem set_seq_union_right_family_monotone[K](a: (Nat, Nat) -> Set[K],
    b: (Nat, Nat) -> Set[K]) {
    (forall(i: Nat) { forall(j: Nat) { a(i, j).subset(b(i, j)) } }) implies
    seq_union(set_seq_union_right_family(a)).subset(seq_union(set_seq_union_right_family(b)))
} by {
    if forall(i: Nat) { forall(j: Nat) { a(i, j).subset(b(i, j)) } } {
        set_sSup_pair_family_monotone(a, b)
        set_sSup_pair_family_eq_seq_union_right(a)
        set_sSup_pair_family_eq_seq_union_right(b)
        set_sSup(set_pair_family(a)).subset(set_sSup(set_pair_family(b)))
        seq_union(set_seq_union_right_family(a)).subset(seq_union(set_seq_union_right_family(b)))
    }
}

/// Pointwise inclusion of natural binary families preserves the other iterated sequence unions.
theorem set_seq_union_left_family_monotone[K](a: (Nat, Nat) -> Set[K],
    b: (Nat, Nat) -> Set[K]) {
    (forall(i: Nat) { forall(j: Nat) { a(i, j).subset(b(i, j)) } }) implies
    seq_union(set_seq_union_left_family(a)).subset(seq_union(set_seq_union_left_family(b)))
} by {
    if forall(i: Nat) { forall(j: Nat) { a(i, j).subset(b(i, j)) } } {
        set_sSup_pair_family_monotone(a, b)
        set_sSup_pair_family_eq_seq_union_left(a)
        set_sSup_pair_family_eq_seq_union_left(b)
        set_sSup(set_pair_family(a)).subset(set_sSup(set_pair_family(b)))
        seq_union(set_seq_union_left_family(a)).subset(seq_union(set_seq_union_left_family(b)))
    }
}

/// Pointwise inclusion of natural binary families preserves iterated sequence intersections.
theorem set_seq_intersection_right_family_monotone[K](a: (Nat, Nat) -> Set[K],
    b: (Nat, Nat) -> Set[K]) {
    (forall(i: Nat) { forall(j: Nat) { a(i, j).subset(b(i, j)) } }) implies
    seq_intersection(set_seq_intersection_right_family(a)).subset(
        seq_intersection(set_seq_intersection_right_family(b)))
} by {
    if forall(i: Nat) { forall(j: Nat) { a(i, j).subset(b(i, j)) } } {
        set_sInf_pair_family_monotone(a, b)
        set_sInf_pair_family_eq_seq_intersection_right(a)
        set_sInf_pair_family_eq_seq_intersection_right(b)
        set_sInf(set_pair_family(a)).subset(set_sInf(set_pair_family(b)))
        seq_intersection(set_seq_intersection_right_family(a)).subset(
            seq_intersection(set_seq_intersection_right_family(b)))
    }
}

/// Pointwise inclusion of natural binary families preserves the other iterated sequence intersections.
theorem set_seq_intersection_left_family_monotone[K](a: (Nat, Nat) -> Set[K],
    b: (Nat, Nat) -> Set[K]) {
    (forall(i: Nat) { forall(j: Nat) { a(i, j).subset(b(i, j)) } }) implies
    seq_intersection(set_seq_intersection_left_family(a)).subset(
        seq_intersection(set_seq_intersection_left_family(b)))
} by {
    if forall(i: Nat) { forall(j: Nat) { a(i, j).subset(b(i, j)) } } {
        set_sInf_pair_family_monotone(a, b)
        set_sInf_pair_family_eq_seq_intersection_left(a)
        set_sInf_pair_family_eq_seq_intersection_left(b)
        set_sInf(set_pair_family(a)).subset(set_sInf(set_pair_family(b)))
        seq_intersection(set_seq_intersection_left_family(a)).subset(
            seq_intersection(set_seq_intersection_left_family(b)))
    }
}

/// Componentwise reindexing of natural binary families gives an iterated sequence union contained in the original one.
theorem set_seq_union_right_reindex_binary_family_subset[K](family: (Nat, Nat) -> Set[K],
    h: Nat -> Nat, k: Nat -> Nat) {
    seq_union(set_seq_union_right_family(set_reindex_binary_family(family, h, k))).subset(
        seq_union(set_seq_union_right_family(family)))
} by {
    set_sSup_reindex_binary_family_subset(family, h, k)
    set_sSup_pair_family_eq_seq_union_right(set_reindex_binary_family(family, h, k))
    set_sSup_pair_family_eq_seq_union_right(family)
    set_sSup(set_pair_family(set_reindex_binary_family(family, h, k))).subset(
        set_sSup(set_pair_family(family)))
    seq_union(set_seq_union_right_family(set_reindex_binary_family(family, h, k))).subset(
        seq_union(set_seq_union_right_family(family)))
}

/// Componentwise reindexing of natural binary families gives the other iterated sequence union contained in the original one.
theorem set_seq_union_left_reindex_binary_family_subset[K](family: (Nat, Nat) -> Set[K],
    h: Nat -> Nat, k: Nat -> Nat) {
    seq_union(set_seq_union_left_family(set_reindex_binary_family(family, h, k))).subset(
        seq_union(set_seq_union_left_family(family)))
} by {
    set_sSup_reindex_binary_family_subset(family, h, k)
    set_sSup_pair_family_eq_seq_union_left(set_reindex_binary_family(family, h, k))
    set_sSup_pair_family_eq_seq_union_left(family)
    set_sSup(set_pair_family(set_reindex_binary_family(family, h, k))).subset(
        set_sSup(set_pair_family(family)))
    seq_union(set_seq_union_left_family(set_reindex_binary_family(family, h, k))).subset(
        seq_union(set_seq_union_left_family(family)))
}

/// The original iterated sequence intersection is contained in a componentwise reindexed one.
theorem set_seq_intersection_right_subset_reindex_binary_family[K](
    family: (Nat, Nat) -> Set[K], h: Nat -> Nat, k: Nat -> Nat
) {
    seq_intersection(set_seq_intersection_right_family(family)).subset(
        seq_intersection(
            set_seq_intersection_right_family(set_reindex_binary_family(family, h, k))))
} by {
    set_sInf_subset_reindex_binary_family(family, h, k)
    set_sInf_pair_family_eq_seq_intersection_right(family)
    set_sInf_pair_family_eq_seq_intersection_right(set_reindex_binary_family(family, h, k))
    set_sInf(set_pair_family(family)).subset(
        set_sInf(set_pair_family(set_reindex_binary_family(family, h, k))))
    seq_intersection(set_seq_intersection_right_family(family)).subset(
        seq_intersection(
            set_seq_intersection_right_family(set_reindex_binary_family(family, h, k))))
}

/// The original other iterated sequence intersection is contained in a componentwise reindexed one.
theorem set_seq_intersection_left_subset_reindex_binary_family[K](
    family: (Nat, Nat) -> Set[K], h: Nat -> Nat, k: Nat -> Nat
) {
    seq_intersection(set_seq_intersection_left_family(family)).subset(
        seq_intersection(
            set_seq_intersection_left_family(set_reindex_binary_family(family, h, k))))
} by {
    set_sInf_subset_reindex_binary_family(family, h, k)
    set_sInf_pair_family_eq_seq_intersection_left(family)
    set_sInf_pair_family_eq_seq_intersection_left(set_reindex_binary_family(family, h, k))
    set_sInf(set_pair_family(family)).subset(
        set_sInf(set_pair_family(set_reindex_binary_family(family, h, k))))
    seq_intersection(set_seq_intersection_left_family(family)).subset(
        seq_intersection(
            set_seq_intersection_left_family(set_reindex_binary_family(family, h, k))))
}

/// Componentwise surjective reindexing preserves iterated sequence unions.
theorem set_seq_union_right_reindex_binary_family_eq_of_surjective[K](
    family: (Nat, Nat) -> Set[K], h: Nat -> Nat, k: Nat -> Nat
) {
    is_surjective_fn(h) and is_surjective_fn(k) implies
    seq_union(set_seq_union_right_family(set_reindex_binary_family(family, h, k))) =
        seq_union(set_seq_union_right_family(family))
} by {
    if is_surjective_fn(h) and is_surjective_fn(k) {
        set_sSup_reindex_binary_family_eq_of_surjective(family, h, k)
        set_sSup_pair_family_eq_seq_union_right(set_reindex_binary_family(family, h, k))
        set_sSup_pair_family_eq_seq_union_right(family)
        set_sSup(set_pair_family(set_reindex_binary_family(family, h, k))) =
            set_sSup(set_pair_family(family))
        seq_union(set_seq_union_right_family(set_reindex_binary_family(family, h, k))) =
            seq_union(set_seq_union_right_family(family))
    }
}

/// Componentwise surjective reindexing preserves the other iterated sequence unions.
theorem set_seq_union_left_reindex_binary_family_eq_of_surjective[K](
    family: (Nat, Nat) -> Set[K], h: Nat -> Nat, k: Nat -> Nat
) {
    is_surjective_fn(h) and is_surjective_fn(k) implies
    seq_union(set_seq_union_left_family(set_reindex_binary_family(family, h, k))) =
        seq_union(set_seq_union_left_family(family))
} by {
    if is_surjective_fn(h) and is_surjective_fn(k) {
        set_sSup_reindex_binary_family_eq_of_surjective(family, h, k)
        set_sSup_pair_family_eq_seq_union_left(set_reindex_binary_family(family, h, k))
        set_sSup_pair_family_eq_seq_union_left(family)
        set_sSup(set_pair_family(set_reindex_binary_family(family, h, k))) =
            set_sSup(set_pair_family(family))
        seq_union(set_seq_union_left_family(set_reindex_binary_family(family, h, k))) =
            seq_union(set_seq_union_left_family(family))
    }
}

/// Componentwise surjective reindexing preserves iterated sequence intersections.
theorem set_seq_intersection_right_reindex_binary_family_eq_of_surjective[K](
    family: (Nat, Nat) -> Set[K], h: Nat -> Nat, k: Nat -> Nat
) {
    is_surjective_fn(h) and is_surjective_fn(k) implies
    seq_intersection(set_seq_intersection_right_family(set_reindex_binary_family(family, h, k))) =
        seq_intersection(set_seq_intersection_right_family(family))
} by {
    if is_surjective_fn(h) and is_surjective_fn(k) {
        set_sInf_reindex_binary_family_eq_of_surjective(family, h, k)
        set_sInf_pair_family_eq_seq_intersection_right(set_reindex_binary_family(family, h, k))
        set_sInf_pair_family_eq_seq_intersection_right(family)
        set_sInf(set_pair_family(set_reindex_binary_family(family, h, k))) =
            set_sInf(set_pair_family(family))
        seq_intersection(set_seq_intersection_right_family(set_reindex_binary_family(family, h, k))) =
            seq_intersection(set_seq_intersection_right_family(family))
    }
}

/// Componentwise surjective reindexing preserves the other iterated sequence intersections.
theorem set_seq_intersection_left_reindex_binary_family_eq_of_surjective[K](
    family: (Nat, Nat) -> Set[K], h: Nat -> Nat, k: Nat -> Nat
) {
    is_surjective_fn(h) and is_surjective_fn(k) implies
    seq_intersection(set_seq_intersection_left_family(set_reindex_binary_family(family, h, k))) =
        seq_intersection(set_seq_intersection_left_family(family))
} by {
    if is_surjective_fn(h) and is_surjective_fn(k) {
        set_sInf_reindex_binary_family_eq_of_surjective(family, h, k)
        set_sInf_pair_family_eq_seq_intersection_left(set_reindex_binary_family(family, h, k))
        set_sInf_pair_family_eq_seq_intersection_left(family)
        set_sInf(set_pair_family(set_reindex_binary_family(family, h, k))) =
            set_sInf(set_pair_family(family))
        seq_intersection(set_seq_intersection_left_family(set_reindex_binary_family(family, h, k))) =
            seq_intersection(set_seq_intersection_left_family(family))
    }
}

/// Componentwise reindexing with sections preserves iterated sequence unions.
theorem set_seq_union_right_reindex_binary_family_eq_of_sections[K](
    family: (Nat, Nat) -> Set[K], h: Nat -> Nat, k: Nat -> Nat,
    g: Nat -> Nat, l: Nat -> Nat
) {
    (forall(i: Nat) { h(g(i)) = i }) and (forall(j: Nat) { k(l(j)) = j }) implies
    seq_union(set_seq_union_right_family(set_reindex_binary_family(family, h, k))) =
        seq_union(set_seq_union_right_family(family))
} by {
    if (forall(i: Nat) { h(g(i)) = i }) and (forall(j: Nat) { k(l(j)) = j }) {
        set_sSup_reindex_binary_family_eq_of_sections(family, h, k, g, l)
        set_sSup_pair_family_eq_seq_union_right(set_reindex_binary_family(family, h, k))
        set_sSup_pair_family_eq_seq_union_right(family)
        set_sSup(set_pair_family(set_reindex_binary_family(family, h, k))) =
            set_sSup(set_pair_family(family))
        seq_union(set_seq_union_right_family(set_reindex_binary_family(family, h, k))) =
            seq_union(set_seq_union_right_family(family))
    }
}

/// Componentwise reindexing with sections preserves the other iterated sequence unions.
theorem set_seq_union_left_reindex_binary_family_eq_of_sections[K](
    family: (Nat, Nat) -> Set[K], h: Nat -> Nat, k: Nat -> Nat,
    g: Nat -> Nat, l: Nat -> Nat
) {
    (forall(i: Nat) { h(g(i)) = i }) and (forall(j: Nat) { k(l(j)) = j }) implies
    seq_union(set_seq_union_left_family(set_reindex_binary_family(family, h, k))) =
        seq_union(set_seq_union_left_family(family))
} by {
    if (forall(i: Nat) { h(g(i)) = i }) and (forall(j: Nat) { k(l(j)) = j }) {
        set_sSup_reindex_binary_family_eq_of_sections(family, h, k, g, l)
        set_sSup_pair_family_eq_seq_union_left(set_reindex_binary_family(family, h, k))
        set_sSup_pair_family_eq_seq_union_left(family)
        set_sSup(set_pair_family(set_reindex_binary_family(family, h, k))) =
            set_sSup(set_pair_family(family))
        seq_union(set_seq_union_left_family(set_reindex_binary_family(family, h, k))) =
            seq_union(set_seq_union_left_family(family))
    }
}

/// Componentwise reindexing with sections preserves iterated sequence intersections.
theorem set_seq_intersection_right_reindex_binary_family_eq_of_sections[K](
    family: (Nat, Nat) -> Set[K], h: Nat -> Nat, k: Nat -> Nat,
    g: Nat -> Nat, l: Nat -> Nat
) {
    (forall(i: Nat) { h(g(i)) = i }) and (forall(j: Nat) { k(l(j)) = j }) implies
    seq_intersection(set_seq_intersection_right_family(set_reindex_binary_family(family, h, k))) =
        seq_intersection(set_seq_intersection_right_family(family))
} by {
    if (forall(i: Nat) { h(g(i)) = i }) and (forall(j: Nat) { k(l(j)) = j }) {
        set_sInf_reindex_binary_family_eq_of_sections(family, h, k, g, l)
        set_sInf_pair_family_eq_seq_intersection_right(set_reindex_binary_family(family, h, k))
        set_sInf_pair_family_eq_seq_intersection_right(family)
        set_sInf(set_pair_family(set_reindex_binary_family(family, h, k))) =
            set_sInf(set_pair_family(family))
        seq_intersection(set_seq_intersection_right_family(set_reindex_binary_family(family, h, k))) =
            seq_intersection(set_seq_intersection_right_family(family))
    }
}

/// Componentwise reindexing with sections preserves the other iterated sequence intersections.
theorem set_seq_intersection_left_reindex_binary_family_eq_of_sections[K](
    family: (Nat, Nat) -> Set[K], h: Nat -> Nat, k: Nat -> Nat,
    g: Nat -> Nat, l: Nat -> Nat
) {
    (forall(i: Nat) { h(g(i)) = i }) and (forall(j: Nat) { k(l(j)) = j }) implies
    seq_intersection(set_seq_intersection_left_family(set_reindex_binary_family(family, h, k))) =
        seq_intersection(set_seq_intersection_left_family(family))
} by {
    if (forall(i: Nat) { h(g(i)) = i }) and (forall(j: Nat) { k(l(j)) = j }) {
        set_sInf_reindex_binary_family_eq_of_sections(family, h, k, g, l)
        set_sInf_pair_family_eq_seq_intersection_left(set_reindex_binary_family(family, h, k))
        set_sInf_pair_family_eq_seq_intersection_left(family)
        set_sInf(set_pair_family(set_reindex_binary_family(family, h, k))) =
            set_sInf(set_pair_family(family))
        seq_intersection(set_seq_intersection_left_family(set_reindex_binary_family(family, h, k))) =
        seq_intersection(set_seq_intersection_left_family(family))
    }
}

/// Componentwise right-inverse reindexing preserves iterated sequence unions.
theorem set_seq_union_right_reindex_binary_family_eq_of_right_inverses[K](
    family: (Nat, Nat) -> Set[K], h: Nat -> Nat, k: Nat -> Nat,
    g: Nat -> Nat, l: Nat -> Nat
) {
    is_right_inverse_fn(h, g) and is_right_inverse_fn(k, l) implies
    seq_union(set_seq_union_right_family(set_reindex_binary_family(family, h, k))) =
        seq_union(set_seq_union_right_family(family))
} by {
    if is_right_inverse_fn(h, g) and is_right_inverse_fn(k, l) {
        set_sSup_reindex_binary_family_eq_of_right_inverses(family, h, k, g, l)
        set_sSup_pair_family_eq_seq_union_right(set_reindex_binary_family(family, h, k))
        set_sSup_pair_family_eq_seq_union_right(family)
        set_sSup(set_pair_family(set_reindex_binary_family(family, h, k))) =
            set_sSup(set_pair_family(family))
        seq_union(set_seq_union_right_family(set_reindex_binary_family(family, h, k))) =
            seq_union(set_seq_union_right_family(family))
    }
}

/// Componentwise right-inverse reindexing preserves the other iterated sequence unions.
theorem set_seq_union_left_reindex_binary_family_eq_of_right_inverses[K](
    family: (Nat, Nat) -> Set[K], h: Nat -> Nat, k: Nat -> Nat,
    g: Nat -> Nat, l: Nat -> Nat
) {
    is_right_inverse_fn(h, g) and is_right_inverse_fn(k, l) implies
    seq_union(set_seq_union_left_family(set_reindex_binary_family(family, h, k))) =
        seq_union(set_seq_union_left_family(family))
} by {
    if is_right_inverse_fn(h, g) and is_right_inverse_fn(k, l) {
        set_sSup_reindex_binary_family_eq_of_right_inverses(family, h, k, g, l)
        set_sSup_pair_family_eq_seq_union_left(set_reindex_binary_family(family, h, k))
        set_sSup_pair_family_eq_seq_union_left(family)
        set_sSup(set_pair_family(set_reindex_binary_family(family, h, k))) =
            set_sSup(set_pair_family(family))
        seq_union(set_seq_union_left_family(set_reindex_binary_family(family, h, k))) =
            seq_union(set_seq_union_left_family(family))
    }
}

/// Componentwise right-inverse reindexing preserves iterated sequence intersections.
theorem set_seq_intersection_right_reindex_binary_family_eq_of_right_inverses[K](
    family: (Nat, Nat) -> Set[K], h: Nat -> Nat, k: Nat -> Nat,
    g: Nat -> Nat, l: Nat -> Nat
) {
    is_right_inverse_fn(h, g) and is_right_inverse_fn(k, l) implies
    seq_intersection(set_seq_intersection_right_family(set_reindex_binary_family(family, h, k))) =
        seq_intersection(set_seq_intersection_right_family(family))
} by {
    if is_right_inverse_fn(h, g) and is_right_inverse_fn(k, l) {
        set_sInf_reindex_binary_family_eq_of_right_inverses(family, h, k, g, l)
        set_sInf_pair_family_eq_seq_intersection_right(set_reindex_binary_family(family, h, k))
        set_sInf_pair_family_eq_seq_intersection_right(family)
        set_sInf(set_pair_family(set_reindex_binary_family(family, h, k))) =
            set_sInf(set_pair_family(family))
        seq_intersection(set_seq_intersection_right_family(set_reindex_binary_family(family, h, k))) =
            seq_intersection(set_seq_intersection_right_family(family))
    }
}

/// Componentwise right-inverse reindexing preserves the other iterated sequence intersections.
theorem set_seq_intersection_left_reindex_binary_family_eq_of_right_inverses[K](
    family: (Nat, Nat) -> Set[K], h: Nat -> Nat, k: Nat -> Nat,
    g: Nat -> Nat, l: Nat -> Nat
) {
    is_right_inverse_fn(h, g) and is_right_inverse_fn(k, l) implies
    seq_intersection(set_seq_intersection_left_family(set_reindex_binary_family(family, h, k))) =
        seq_intersection(set_seq_intersection_left_family(family))
} by {
    if is_right_inverse_fn(h, g) and is_right_inverse_fn(k, l) {
        set_sInf_reindex_binary_family_eq_of_right_inverses(family, h, k, g, l)
        set_sInf_pair_family_eq_seq_intersection_left(set_reindex_binary_family(family, h, k))
        set_sInf_pair_family_eq_seq_intersection_left(family)
        set_sInf(set_pair_family(set_reindex_binary_family(family, h, k))) =
            set_sInf(set_pair_family(family))
        seq_intersection(set_seq_intersection_left_family(set_reindex_binary_family(family, h, k))) =
            seq_intersection(set_seq_intersection_left_family(family))
    }
}

/// Componentwise bijective reindexing preserves iterated sequence unions.
theorem set_seq_union_right_reindex_binary_family_eq_of_bijection[K](
    family: (Nat, Nat) -> Set[K], h: Nat -> Nat, k: Nat -> Nat
) {
    is_bijection_fn(h) and is_bijection_fn(k) implies
    seq_union(set_seq_union_right_family(set_reindex_binary_family(family, h, k))) =
        seq_union(set_seq_union_right_family(family))
} by {
    if is_bijection_fn(h) and is_bijection_fn(k) {
        set_sSup_reindex_binary_family_eq_of_bijection(family, h, k)
        set_sSup_pair_family_eq_seq_union_right(set_reindex_binary_family(family, h, k))
        set_sSup_pair_family_eq_seq_union_right(family)
        set_sSup(set_pair_family(set_reindex_binary_family(family, h, k))) =
            set_sSup(set_pair_family(family))
        seq_union(set_seq_union_right_family(set_reindex_binary_family(family, h, k))) =
            seq_union(set_seq_union_right_family(family))
    }
}

/// Componentwise bijective reindexing preserves the other iterated sequence unions.
theorem set_seq_union_left_reindex_binary_family_eq_of_bijection[K](
    family: (Nat, Nat) -> Set[K], h: Nat -> Nat, k: Nat -> Nat
) {
    is_bijection_fn(h) and is_bijection_fn(k) implies
    seq_union(set_seq_union_left_family(set_reindex_binary_family(family, h, k))) =
        seq_union(set_seq_union_left_family(family))
} by {
    if is_bijection_fn(h) and is_bijection_fn(k) {
        set_sSup_reindex_binary_family_eq_of_bijection(family, h, k)
        set_sSup_pair_family_eq_seq_union_left(set_reindex_binary_family(family, h, k))
        set_sSup_pair_family_eq_seq_union_left(family)
        set_sSup(set_pair_family(set_reindex_binary_family(family, h, k))) =
            set_sSup(set_pair_family(family))
        seq_union(set_seq_union_left_family(set_reindex_binary_family(family, h, k))) =
            seq_union(set_seq_union_left_family(family))
    }
}

/// Componentwise bijective reindexing preserves iterated sequence intersections.
theorem set_seq_intersection_right_reindex_binary_family_eq_of_bijection[K](
    family: (Nat, Nat) -> Set[K], h: Nat -> Nat, k: Nat -> Nat
) {
    is_bijection_fn(h) and is_bijection_fn(k) implies
    seq_intersection(set_seq_intersection_right_family(set_reindex_binary_family(family, h, k))) =
        seq_intersection(set_seq_intersection_right_family(family))
} by {
    if is_bijection_fn(h) and is_bijection_fn(k) {
        set_sInf_reindex_binary_family_eq_of_bijection(family, h, k)
        set_sInf_pair_family_eq_seq_intersection_right(set_reindex_binary_family(family, h, k))
        set_sInf_pair_family_eq_seq_intersection_right(family)
        set_sInf(set_pair_family(set_reindex_binary_family(family, h, k))) =
            set_sInf(set_pair_family(family))
        seq_intersection(set_seq_intersection_right_family(set_reindex_binary_family(family, h, k))) =
            seq_intersection(set_seq_intersection_right_family(family))
    }
}

/// Componentwise bijective reindexing preserves the other iterated sequence intersections.
theorem set_seq_intersection_left_reindex_binary_family_eq_of_bijection[K](
    family: (Nat, Nat) -> Set[K], h: Nat -> Nat, k: Nat -> Nat
) {
    is_bijection_fn(h) and is_bijection_fn(k) implies
    seq_intersection(set_seq_intersection_left_family(set_reindex_binary_family(family, h, k))) =
        seq_intersection(set_seq_intersection_left_family(family))
} by {
    if is_bijection_fn(h) and is_bijection_fn(k) {
        set_sInf_reindex_binary_family_eq_of_bijection(family, h, k)
        set_sInf_pair_family_eq_seq_intersection_left(set_reindex_binary_family(family, h, k))
        set_sInf_pair_family_eq_seq_intersection_left(family)
        set_sInf(set_pair_family(set_reindex_binary_family(family, h, k))) =
            set_sInf(set_pair_family(family))
        seq_intersection(set_seq_intersection_left_family(set_reindex_binary_family(family, h, k))) =
            seq_intersection(set_seq_intersection_left_family(family))
    }
}

/// A preimage of an iterated sequence union is contained in a set exactly when every term preimage is contained in it.
theorem set_preimage_seq_union_right_family_subset_iff[T, U](f: T -> U,
    family: (Nat, Nat) -> Set[U], s: Set[T]) {
    set_preimage(f, seq_union(set_seq_union_right_family(family))).subset(s) =
    forall(i: Nat) {
        forall(j: Nat) {
            set_preimage_binary_family(f, family, i, j).subset(s)
        }
    }
} by {
    set_preimage_seq_union_right_family(f, family)
    set_seq_union_right_family_subset_iff(set_preimage_binary_family(f, family), s)
    set_preimage(f, seq_union(set_seq_union_right_family(family))).subset(s) =
    forall(i: Nat) {
        forall(j: Nat) {
            set_preimage_binary_family(f, family, i, j).subset(s)
        }
    }
}

/// A preimage of the other iterated sequence union is contained in a set exactly when every term preimage is contained in it.
theorem set_preimage_seq_union_left_family_subset_iff[T, U](f: T -> U,
    family: (Nat, Nat) -> Set[U], s: Set[T]) {
    set_preimage(f, seq_union(set_seq_union_left_family(family))).subset(s) =
    forall(i: Nat) {
        forall(j: Nat) {
            set_preimage_binary_family(f, family, i, j).subset(s)
        }
    }
} by {
    set_preimage_seq_union_left_family(f, family)
    set_seq_union_left_family_subset_iff(set_preimage_binary_family(f, family), s)
    set_preimage(f, seq_union(set_seq_union_left_family(family))).subset(s) =
    forall(i: Nat) {
        forall(j: Nat) {
            set_preimage_binary_family(f, family, i, j).subset(s)
        }
    }
}

/// A set is contained in a preimage of an iterated sequence intersection exactly when it is contained in every term preimage.
theorem set_subset_preimage_seq_intersection_right_family_iff[T, U](s: Set[T],
    f: T -> U, family: (Nat, Nat) -> Set[U]) {
    s.subset(set_preimage(f, seq_intersection(set_seq_intersection_right_family(family)))) =
    forall(i: Nat) {
        forall(j: Nat) {
            s.subset(set_preimage_binary_family(f, family, i, j))
        }
    }
} by {
    set_preimage_seq_intersection_right_family(f, family)
    set_subset_seq_intersection_right_family_iff(s, set_preimage_binary_family(f, family))
    s.subset(set_preimage(f, seq_intersection(set_seq_intersection_right_family(family)))) =
    forall(i: Nat) {
        forall(j: Nat) {
            s.subset(set_preimage_binary_family(f, family, i, j))
        }
    }
}

/// A set is contained in a preimage of the other iterated sequence intersection exactly when it is contained in every term preimage.
theorem set_subset_preimage_seq_intersection_left_family_iff[T, U](s: Set[T],
    f: T -> U, family: (Nat, Nat) -> Set[U]) {
    s.subset(set_preimage(f, seq_intersection(set_seq_intersection_left_family(family)))) =
    forall(i: Nat) {
        forall(j: Nat) {
            s.subset(set_preimage_binary_family(f, family, i, j))
        }
    }
} by {
    set_preimage_seq_intersection_left_family(f, family)
    set_subset_seq_intersection_left_family_iff(s, set_preimage_binary_family(f, family))
    s.subset(set_preimage(f, seq_intersection(set_seq_intersection_left_family(family)))) =
    forall(i: Nat) {
        forall(j: Nat) {
            s.subset(set_preimage_binary_family(f, family, i, j))
        }
    }
}

/// Image of an iterated sequence union is contained in a target exactly when every indexed member maps into it.
theorem set_image_seq_union_right_family_subset_iff[T, U](
    family: (Nat, Nat) -> Set[T], s: Set[U], f: T -> U
) {
    set_image(seq_union(set_seq_union_right_family(family)), f).subset(s) =
    forall(i: Nat) {
        forall(j: Nat) {
            family(i, j).subset(set_preimage(f, s))
        }
    }
} by {
    set_image_seq_union_right_family(family, f)
    set_seq_union_right_family_subset_iff(set_image_binary_family(family, f), s)
    set_image_subset_iff_subset_preimage(seq_union(set_seq_union_right_family(family)), s, f)
    set_seq_union_right_family_subset_iff(family, set_preimage(f, s))
    set_image(seq_union(set_seq_union_right_family(family)), f).subset(s) =
    forall(i: Nat) {
        forall(j: Nat) {
            family(i, j).subset(set_preimage(f, s))
        }
    }
}

/// Image of the other iterated sequence union is contained in a target exactly when every indexed member maps into it.
theorem set_image_seq_union_left_family_subset_iff[T, U](
    family: (Nat, Nat) -> Set[T], s: Set[U], f: T -> U
) {
    set_image(seq_union(set_seq_union_left_family(family)), f).subset(s) =
    forall(i: Nat) {
        forall(j: Nat) {
            family(i, j).subset(set_preimage(f, s))
        }
    }
} by {
    set_image_seq_union_left_family(family, f)
    set_seq_union_left_family_subset_iff(set_image_binary_family(family, f), s)
    set_image_subset_iff_subset_preimage(seq_union(set_seq_union_left_family(family)), s, f)
    set_seq_union_left_family_subset_iff(family, set_preimage(f, s))
    set_image(seq_union(set_seq_union_left_family(family)), f).subset(s) =
    forall(i: Nat) {
        forall(j: Nat) {
            family(i, j).subset(set_preimage(f, s))
        }
    }
}

/// Under a bijective map, images preserve an iterated sequence intersection.
theorem set_image_seq_intersection_right_family_of_bijection[T: Inhabited, U](
    family: (Nat, Nat) -> Set[T], f: T -> U
) {
    is_bijection_fn(f) implies
    set_image(seq_intersection(set_seq_intersection_right_family(family)), f) =
        seq_intersection(set_seq_intersection_right_family(set_image_binary_family(family, f)))
} by {
    if is_bijection_fn(f) {
        set_sInf_pair_family_eq_seq_intersection_right(family)
        set_sInf_pair_family_eq_seq_intersection_right(set_image_binary_family(family, f))
        set_image_sInf_pair_family_of_bijection(family, f)
        set_image(set_sInf(set_pair_family(family)), f) =
            set_sInf(set_pair_family(set_image_binary_family(family, f)))
        set_image(seq_intersection(set_seq_intersection_right_family(family)), f) =
            seq_intersection(
                set_seq_intersection_right_family(set_image_binary_family(family, f)))
    }
}

/// Under a bijective map, images preserve the other iterated sequence intersection.
theorem set_image_seq_intersection_left_family_of_bijection[T: Inhabited, U](
    family: (Nat, Nat) -> Set[T], f: T -> U
) {
    is_bijection_fn(f) implies
    set_image(seq_intersection(set_seq_intersection_left_family(family)), f) =
        seq_intersection(set_seq_intersection_left_family(set_image_binary_family(family, f)))
} by {
    if is_bijection_fn(f) {
        set_sInf_pair_family_eq_seq_intersection_left(family)
        set_sInf_pair_family_eq_seq_intersection_left(set_image_binary_family(family, f))
        set_image_sInf_pair_family_of_bijection(family, f)
        set_image(set_sInf(set_pair_family(family)), f) =
            set_sInf(set_pair_family(set_image_binary_family(family, f)))
        set_image(seq_intersection(set_seq_intersection_left_family(family)), f) =
            seq_intersection(
                set_seq_intersection_left_family(set_image_binary_family(family, f)))
    }
}

/// A set is contained in an injective image of an iterated sequence intersection exactly when it is contained in every member image.
theorem set_subset_image_seq_intersection_right_family_iff_of_injective[T, U](
    s: Set[U], family: (Nat, Nat) -> Set[T], f: T -> U
) {
    is_injective_fn(f) implies
    s.subset(set_image(seq_intersection(set_seq_intersection_right_family(family)), f)) =
    forall(i: Nat) {
        forall(j: Nat) {
            s.subset(set_image_binary_family(family, f, i, j))
        }
    }
} by {
    if is_injective_fn(f) {
        set_image_seq_intersection_right_family_of_injective(family, f)
        set_subset_seq_intersection_right_family_iff(s, set_image_binary_family(family, f))
        s.subset(set_image(seq_intersection(set_seq_intersection_right_family(family)), f)) =
        forall(i: Nat) {
            forall(j: Nat) {
                s.subset(set_image_binary_family(family, f, i, j))
            }
        }
    }
}

/// A set is contained in an injective image of the other iterated sequence intersection exactly when it is contained in every member image.
theorem set_subset_image_seq_intersection_left_family_iff_of_injective[T, U](
    s: Set[U], family: (Nat, Nat) -> Set[T], f: T -> U
) {
    is_injective_fn(f) implies
    s.subset(set_image(seq_intersection(set_seq_intersection_left_family(family)), f)) =
    forall(i: Nat) {
        forall(j: Nat) {
            s.subset(set_image_binary_family(family, f, i, j))
        }
    }
} by {
    if is_injective_fn(f) {
        set_image_seq_intersection_left_family_of_injective(family, f)
        set_subset_seq_intersection_left_family_iff(s, set_image_binary_family(family, f))
        s.subset(set_image(seq_intersection(set_seq_intersection_left_family(family)), f)) =
        forall(i: Nat) {
            forall(j: Nat) {
                s.subset(set_image_binary_family(family, f, i, j))
            }
        }
    }
}

/// A set is contained in a bijective image of an iterated sequence intersection exactly when it is contained in every member image.
theorem set_subset_image_seq_intersection_right_family_iff_of_bijection[T: Inhabited, U](
    s: Set[U], family: (Nat, Nat) -> Set[T], f: T -> U
) {
    is_bijection_fn(f) implies
    s.subset(set_image(seq_intersection(set_seq_intersection_right_family(family)), f)) =
    forall(i: Nat) {
        forall(j: Nat) {
            s.subset(set_image_binary_family(family, f, i, j))
        }
    }
} by {
    if is_bijection_fn(f) {
        set_image_seq_intersection_right_family_of_bijection(family, f)
        set_subset_seq_intersection_right_family_iff(s, set_image_binary_family(family, f))
        s.subset(set_image(seq_intersection(set_seq_intersection_right_family(family)), f)) =
        forall(i: Nat) {
            forall(j: Nat) {
                s.subset(set_image_binary_family(family, f, i, j))
            }
        }
    }
}

/// A set is contained in a bijective image of the other iterated sequence intersection exactly when it is contained in every member image.
theorem set_subset_image_seq_intersection_left_family_iff_of_bijection[T: Inhabited, U](
    s: Set[U], family: (Nat, Nat) -> Set[T], f: T -> U
) {
    is_bijection_fn(f) implies
    s.subset(set_image(seq_intersection(set_seq_intersection_left_family(family)), f)) =
    forall(i: Nat) {
        forall(j: Nat) {
            s.subset(set_image_binary_family(family, f, i, j))
        }
    }
} by {
    if is_bijection_fn(f) {
        set_image_seq_intersection_left_family_of_bijection(family, f)
        set_subset_seq_intersection_left_family_iff(s, set_image_binary_family(family, f))
        s.subset(set_image(seq_intersection(set_seq_intersection_left_family(family)), f)) =
        forall(i: Nat) {
            forall(j: Nat) {
                s.subset(set_image_binary_family(family, f, i, j))
            }
        }
    }
}

/// Every indexed member is contained in the right-associated iterated sequence union.
theorem set_seq_union_right_family_subset_of_member[K](
    family: (Nat, Nat) -> Set[K], i: Nat, j: Nat
) {
    family(i, j).subset(seq_union(set_seq_union_right_family(family)))
} by {
    set_pair_family_subset_sSup(family, i, j)
    set_sSup_pair_family_eq_seq_union_right(family)
    family(i, j).subset(set_sSup(set_pair_family(family)))
    family(i, j).subset(seq_union(set_seq_union_right_family(family)))
}

/// Every indexed member is contained in the left-associated iterated sequence union.
theorem set_seq_union_left_family_subset_of_member[K](
    family: (Nat, Nat) -> Set[K], i: Nat, j: Nat
) {
    family(i, j).subset(seq_union(set_seq_union_left_family(family)))
} by {
    set_pair_family_subset_sSup(family, i, j)
    set_sSup_pair_family_eq_seq_union_left(family)
    family(i, j).subset(set_sSup(set_pair_family(family)))
    family(i, j).subset(seq_union(set_seq_union_left_family(family)))
}

/// The right-associated iterated sequence intersection is contained in every indexed member.
theorem set_seq_intersection_right_family_subset_member[K](
    family: (Nat, Nat) -> Set[K], i: Nat, j: Nat
) {
    seq_intersection(set_seq_intersection_right_family(family)).subset(family(i, j))
} by {
    set_sInf_pair_family_subset_family(family, i, j)
    set_sInf_pair_family_eq_seq_intersection_right(family)
    set_sInf(set_pair_family(family)).subset(family(i, j))
    seq_intersection(set_seq_intersection_right_family(family)).subset(family(i, j))
}

/// The left-associated iterated sequence intersection is contained in every indexed member.
theorem set_seq_intersection_left_family_subset_member[K](
    family: (Nat, Nat) -> Set[K], i: Nat, j: Nat
) {
    seq_intersection(set_seq_intersection_left_family(family)).subset(family(i, j))
} by {
    set_sInf_pair_family_subset_family(family, i, j)
    set_sInf_pair_family_eq_seq_intersection_left(family)
    set_sInf(set_pair_family(family)).subset(family(i, j))
    seq_intersection(set_seq_intersection_left_family(family)).subset(family(i, j))
}

/// The right-associated iterated sequence union is the least upper bound of the binary family.
theorem set_seq_union_right_family_is_lub[K](family: (Nat, Nat) -> Set[K],
    s: Set[K]) {
    (forall(i: Nat) { forall(j: Nat) {
        family(i, j).subset(seq_union(set_seq_union_right_family(family)))
    } }) and
    (seq_union(set_seq_union_right_family(family)).subset(s) = forall(i: Nat) {
        forall(j: Nat) {
            family(i, j).subset(s)
        }
    })
} by {
    forall(i: Nat) {
        forall(j: Nat) {
            set_seq_union_right_family_subset_of_member(family, i, j)
        }
    }
    set_seq_union_right_family_subset_iff(family, s)
}

/// The left-associated iterated sequence union is the least upper bound of the binary family.
theorem set_seq_union_left_family_is_lub[K](family: (Nat, Nat) -> Set[K],
    s: Set[K]) {
    (forall(i: Nat) { forall(j: Nat) {
        family(i, j).subset(seq_union(set_seq_union_left_family(family)))
    } }) and
    (seq_union(set_seq_union_left_family(family)).subset(s) = forall(i: Nat) {
        forall(j: Nat) {
            family(i, j).subset(s)
        }
    })
} by {
    forall(i: Nat) {
        forall(j: Nat) {
            set_seq_union_left_family_subset_of_member(family, i, j)
        }
    }
    set_seq_union_left_family_subset_iff(family, s)
}

/// The right-associated iterated sequence intersection is the greatest lower bound of the binary family.
theorem set_seq_intersection_right_family_is_glb[K](s: Set[K],
    family: (Nat, Nat) -> Set[K]) {
    (forall(i: Nat) { forall(j: Nat) {
        seq_intersection(set_seq_intersection_right_family(family)).subset(family(i, j))
    } }) and
    (s.subset(seq_intersection(set_seq_intersection_right_family(family))) =
    forall(i: Nat) {
        forall(j: Nat) {
            s.subset(family(i, j))
        }
    })
} by {
    forall(i: Nat) {
        forall(j: Nat) {
            set_seq_intersection_right_family_subset_member(family, i, j)
        }
    }
    set_subset_seq_intersection_right_family_iff(s, family)
}

/// The left-associated iterated sequence intersection is the greatest lower bound of the binary family.
theorem set_seq_intersection_left_family_is_glb[K](s: Set[K],
    family: (Nat, Nat) -> Set[K]) {
    (forall(i: Nat) { forall(j: Nat) {
        seq_intersection(set_seq_intersection_left_family(family)).subset(family(i, j))
    } }) and
    (s.subset(seq_intersection(set_seq_intersection_left_family(family))) =
    forall(i: Nat) {
        forall(j: Nat) {
            s.subset(family(i, j))
        }
    })
} by {
    forall(i: Nat) {
        forall(j: Nat) {
            set_seq_intersection_left_family_subset_member(family, i, j)
        }
    }
    set_subset_seq_intersection_left_family_iff(s, family)
}

