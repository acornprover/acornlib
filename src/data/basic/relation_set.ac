/// Set-theoretic operations induced by unbundled relations.
///
/// This module bridges relations `(A, B) -> Bool` with `Set` operations.

from data.basic.set import Set, set_ext, subset_contains, empty_set_contains_eq,
    universal_set_contains_eq, intersection_contains_eq, intersection_contains_intro
from data.basic.relation_basic import eq_relation, relation_converse, relation_compose, relation_subset,
    relation_subset_step, relation_compose_intro, relation_compose_has_witness

/// True if `y` lies in the image of `s` under relation `r`.
define relation_image_contains[A, B](r: (A, B) -> Bool, s: Set[A], y: B) -> Bool {
    exists(x: A) {
        s.contains(x) and r(x, y)
    }
}

/// The image of a source set under an unbundled relation.
define relation_image[A, B](r: (A, B) -> Bool, s: Set[A]) -> Set[B] {
    Set[B].new(relation_image_contains(r, s))
}

/// Membership in a relational image is the image predicate.
theorem relation_image_contains_eq[A, B](r: (A, B) -> Bool, s: Set[A], y: B) {
    relation_image(r, s).contains(y) = relation_image_contains(r, s, y)
}

/// A related point of the source set lies in the relational image.
theorem relation_image_intro[A, B](r: (A, B) -> Bool, s: Set[A], x: A, y: B) {
    s.contains(x) and r(x, y) implies relation_image(r, s).contains(y)
} by {
    if s.contains(x) and r(x, y) {
        exists(w: A) {
            w = x and s.contains(w) and r(w, y)
        }
        relation_image_contains(r, s, y)
    }
}

/// Membership in a relational image yields a source witness.
theorem relation_image_has_witness[A, B](r: (A, B) -> Bool, s: Set[A], y: B) {
    relation_image(r, s).contains(y) implies exists(x: A) {
        s.contains(x) and r(x, y)
    }
} by {
    if relation_image(r, s).contains(y) {
        relation_image_contains_eq(r, s, y)
        relation_image_contains(r, s, y)
    }
}

/// True if `x` maps by relation `r` into the target set `t`.
define relation_preimage_contains[A, B](r: (A, B) -> Bool, t: Set[B], x: A) -> Bool {
    exists(y: B) {
        t.contains(y) and r(x, y)
    }
}

/// The preimage of a target set under an unbundled relation.
define relation_preimage[A, B](r: (A, B) -> Bool, t: Set[B]) -> Set[A] {
    Set[A].new(relation_preimage_contains(r, t))
}

/// Membership in a relational preimage is the preimage predicate.
theorem relation_preimage_contains_eq[A, B](r: (A, B) -> Bool, t: Set[B], x: A) {
    relation_preimage(r, t).contains(x) = relation_preimage_contains(r, t, x)
}

/// A relation edge into a target-set point gives preimage membership.
theorem relation_preimage_intro[A, B](r: (A, B) -> Bool, t: Set[B], x: A, y: B) {
    t.contains(y) and r(x, y) implies relation_preimage(r, t).contains(x)
} by {
    if t.contains(y) and r(x, y) {
        exists(w: B) {
            w = y and t.contains(w) and r(x, w)
        }
        relation_preimage_contains(r, t, x)
    }
}

/// Membership in a relational preimage yields a target witness.
theorem relation_preimage_has_witness[A, B](r: (A, B) -> Bool, t: Set[B], x: A) {
    relation_preimage(r, t).contains(x) implies exists(y: B) {
        t.contains(y) and r(x, y)
    }
} by {
    if relation_preimage(r, t).contains(x) {
        relation_preimage_contains_eq(r, t, x)
        relation_preimage_contains(r, t, x)
    }
}

/// The domain of a relation, expressed as the converse image of the universal target set.
define relation_domain[A, B](r: (A, B) -> Bool) -> Set[A] {
    relation_image(relation_converse(r), Set[B].universal_set)
}

/// The range of a relation, expressed as the image of the universal source set.
define relation_range[A, B](r: (A, B) -> Bool) -> Set[B] {
    relation_image(r, Set[A].universal_set)
}

/// Restrict a relation to a source set.
define relation_restrict_source[A, B](r: (A, B) -> Bool, s: Set[A], x: A, y: B) -> Bool {
    s.contains(x) and r(x, y)
}

/// Corestrict a relation to a target set.
define relation_corestrict_target[A, B](r: (A, B) -> Bool, t: Set[B], x: A, y: B) -> Bool {
    r(x, y) and t.contains(y)
}

/// The relational image is monotone in the input set.
theorem relation_image_monotone_set[A, B](r: (A, B) -> Bool, s: Set[A], t: Set[A]) {
    s.subset(t) implies relation_image(r, s).subset(relation_image(r, t))
} by {
    if s.subset(t) {
        forall(y: B) {
            if relation_image(r, s).contains(y) {
                relation_image_has_witness(r, s, y)
                let x: A satisfy {
                    s.contains(x) and r(x, y)
                }
                subset_contains(s, t, x)
                t.contains(x)
                relation_image_intro(r, t, x, y)
                relation_image(r, t).contains(y)
            }
        }
    }
}

/// The relational preimage is monotone in the target set.
theorem relation_preimage_monotone_set[A, B](r: (A, B) -> Bool, s: Set[B], t: Set[B]) {
    s.subset(t) implies relation_preimage(r, s).subset(relation_preimage(r, t))
} by {
    if s.subset(t) {
        forall(x: A) {
            if relation_preimage(r, s).contains(x) {
                relation_preimage_has_witness(r, s, x)
                let y: B satisfy {
                    s.contains(y) and r(x, y)
                }
                subset_contains(s, t, y)
                t.contains(y)
                relation_preimage_intro(r, t, x, y)
                relation_preimage(r, t).contains(x)
            }
        }
    }
}

/// The relational image is monotone in the relation.
theorem relation_image_monotone_relation[A, B](r: (A, B) -> Bool, s: (A, B) -> Bool, a: Set[A]) {
    relation_subset(r, s) implies relation_image(r, a).subset(relation_image(s, a))
} by {
    if relation_subset(r, s) {
        forall(y: B) {
            if relation_image(r, a).contains(y) {
                relation_image_has_witness(r, a, y)
                let x: A satisfy {
                    a.contains(x) and r(x, y)
                }
                relation_subset_step(r, s, x, y)
                s(x, y)
                relation_image_intro(s, a, x, y)
                relation_image(s, a).contains(y)
            }
        }
    }
}

/// The relational preimage is monotone in the relation.
theorem relation_preimage_monotone_relation[A, B](r: (A, B) -> Bool, s: (A, B) -> Bool, a: Set[B]) {
    relation_subset(r, s) implies relation_preimage(r, a).subset(relation_preimage(s, a))
} by {
    if relation_subset(r, s) {
        forall(x: A) {
            if relation_preimage(r, a).contains(x) {
                relation_preimage_has_witness(r, a, x)
                let y: B satisfy {
                    a.contains(y) and r(x, y)
                }
                relation_subset_step(r, s, x, y)
                s(x, y)
                relation_preimage_intro(s, a, x, y)
                relation_preimage(s, a).contains(x)
            }
        }
    }
}

/// Relational preimage is relational image under the converse relation.
theorem relation_preimage_eq_image_converse[A, B](r: (A, B) -> Bool, t: Set[B]) {
    relation_preimage(r, t) = relation_image(relation_converse(r), t)
} by {
    let u = relation_preimage(r, t)
    let v = relation_image(relation_converse(r), t)
    forall(x: A) {
        if u.contains(x) {
            relation_preimage_has_witness(r, t, x)
            let y: B satisfy {
                t.contains(y) and r(x, y)
            }
            relation_converse(r, y, x)
            relation_image_intro(relation_converse(r), t, y, x)
            v.contains(x)
        }
        if v.contains(x) {
            relation_image_has_witness(relation_converse(r), t, x)
            let y: B satisfy {
                t.contains(y) and relation_converse(r, y, x)
            }
            r(x, y)
            relation_preimage_intro(r, t, x, y)
            u.contains(x)
        }
        u.contains(x) = v.contains(x)
    }
    set_ext(u, v)
}

/// Relational image is relational preimage under the converse relation.
theorem relation_image_eq_preimage_converse[A, B](r: (A, B) -> Bool, s: Set[A]) {
    relation_image(r, s) = relation_preimage(relation_converse(r), s)
} by {
    let u = relation_image(r, s)
    let v = relation_preimage(relation_converse(r), s)
    forall(y: B) {
        if u.contains(y) {
            relation_image_has_witness(r, s, y)
            let x: A satisfy {
                s.contains(x) and r(x, y)
            }
            relation_converse(r, y, x)
            relation_preimage_intro(relation_converse(r), s, y, x)
            v.contains(y)
        }
        if v.contains(y) {
            relation_preimage_has_witness(relation_converse(r), s, y)
            let x: A satisfy {
                s.contains(x) and relation_converse(r, y, x)
            }
            r(x, y)
            relation_image_intro(r, s, x, y)
            u.contains(y)
        }
        u.contains(y) = v.contains(y)
    }
    set_ext(u, v)
}

/// Domain membership is existence of an outgoing related target.
theorem relation_domain_contains_eq[A, B](r: (A, B) -> Bool, x: A) {
    relation_domain(r).contains(x) = exists(y: B) {
        r(x, y)
    }
} by {
    if relation_domain(r).contains(x) {
        relation_image_has_witness(relation_converse(r), Set[B].universal_set, x)
        let y: B satisfy {
            Set[B].universal_set.contains(y) and relation_converse(r, y, x)
        }
        r(x, y)
        exists(z: B) {
            z = y and r(x, z)
        }
    }
    if exists(y: B) { r(x, y) } {
        let y: B satisfy {
            r(x, y)
        }
        universal_set_contains_eq[B](y)
        Set[B].universal_set.contains(y)
        relation_converse(r, y, x)
        relation_image_intro(relation_converse(r), Set[B].universal_set, y, x)
        relation_domain(r).contains(x)
    }
    relation_domain(r).contains(x) = exists(y: B) {
        r(x, y)
    }
}

/// Range membership is existence of an incoming related source.
theorem relation_range_contains_eq[A, B](r: (A, B) -> Bool, y: B) {
    relation_range(r).contains(y) = exists(x: A) {
        r(x, y)
    }
} by {
    if relation_range(r).contains(y) {
        relation_image_has_witness(r, Set[A].universal_set, y)
        let x: A satisfy {
            Set[A].universal_set.contains(x) and r(x, y)
        }
        exists(z: A) {
            z = x and r(z, y)
        }
    }
    if exists(x: A) { r(x, y) } {
        let x: A satisfy {
            r(x, y)
        }
        universal_set_contains_eq[A](x)
        Set[A].universal_set.contains(x)
        relation_image_intro(r, Set[A].universal_set, x, y)
        relation_range(r).contains(y)
    }
    relation_range(r).contains(y) = exists(x: A) {
        r(x, y)
    }
}

/// The preimage of the universal set is the domain of the relation.
theorem relation_preimage_universal_eq_domain[A, B](r: (A, B) -> Bool) {
    relation_preimage(r, Set[B].universal_set) = relation_domain(r)
} by {
    let u = relation_preimage(r, Set[B].universal_set)
    let v = relation_domain(r)
    relation_preimage_eq_image_converse(r, Set[B].universal_set)
    u = relation_image(relation_converse(r), Set[B].universal_set)
    v = relation_image(relation_converse(r), Set[B].universal_set)
    u = v
}

/// The image of the empty set is empty.
theorem relation_image_empty[A, B](r: (A, B) -> Bool) {
    relation_image(r, Set[A].empty_set) = Set[B].empty_set
} by {
    let u = relation_image(r, Set[A].empty_set)
    let v = Set[B].empty_set
    forall(y: B) {
        if u.contains(y) {
            relation_image_has_witness(r, Set[A].empty_set, y)
            let x: A satisfy {
                Set[A].empty_set.contains(x) and r(x, y)
            }
            empty_set_contains_eq[A](x)
            false
        }
        empty_set_contains_eq[B](y)
        u.contains(y) = v.contains(y)
    }
    set_ext(u, v)
}

/// The preimage of the empty set is empty.
theorem relation_preimage_empty[A, B](r: (A, B) -> Bool) {
    relation_preimage(r, Set[B].empty_set) = Set[A].empty_set
} by {
    let u = relation_preimage(r, Set[B].empty_set)
    let v = Set[A].empty_set
    forall(x: A) {
        if u.contains(x) {
            relation_preimage_has_witness(r, Set[B].empty_set, x)
            let y: B satisfy {
                Set[B].empty_set.contains(y) and r(x, y)
            }
            empty_set_contains_eq[B](y)
            false
        }
        empty_set_contains_eq[A](x)
        u.contains(x) = v.contains(x)
    }
    set_ext(u, v)
}

/// Every image is contained in the range of its relation.
theorem relation_image_subset_range[A, B](r: (A, B) -> Bool, s: Set[A]) {
    relation_image(r, s).subset(relation_range(r))
} by {
    forall(y: B) {
        if relation_image(r, s).contains(y) {
            relation_image_has_witness(r, s, y)
            let x: A satisfy {
                s.contains(x) and r(x, y)
            }
            universal_set_contains_eq[A](x)
            Set[A].universal_set.contains(x)
            relation_image_intro(r, Set[A].universal_set, x, y)
            relation_range(r).contains(y)
        }
    }
}

/// Every preimage is contained in the domain of its relation.
theorem relation_preimage_subset_domain[A, B](r: (A, B) -> Bool, t: Set[B]) {
    relation_preimage(r, t).subset(relation_domain(r))
} by {
    forall(x: A) {
        if relation_preimage(r, t).contains(x) {
            relation_preimage_has_witness(r, t, x)
            let y: B satisfy {
                t.contains(y) and r(x, y)
            }
            universal_set_contains_eq[B](y)
            Set[B].universal_set.contains(y)
            relation_converse(r, y, x)
            relation_image_intro(relation_converse(r), Set[B].universal_set, y, x)
            relation_domain(r).contains(x)
        }
    }
}

/// The equality relation maps a set to itself.
theorem relation_image_eq_relation[A](s: Set[A]) {
    relation_image(eq_relation[A], s) = s
} by {
    let u = relation_image(eq_relation[A], s)
    let v = s
    forall(y: A) {
        if u.contains(y) {
            relation_image_has_witness(eq_relation[A], s, y)
            let x: A satisfy {
                s.contains(x) and eq_relation[A](x, y)
            }
            x = y
            v.contains(y)
        }
        if v.contains(y) {
            eq_relation[A](y, y)
            relation_image_intro(eq_relation[A], s, y, y)
            u.contains(y)
        }
        u.contains(y) = v.contains(y)
    }
    set_ext(u, v)
}

/// The equality relation pulls a set back to itself.
theorem relation_preimage_eq_relation[A](s: Set[A]) {
    relation_preimage(eq_relation[A], s) = s
} by {
    let u = relation_preimage(eq_relation[A], s)
    let v = s
    forall(x: A) {
        if u.contains(x) {
            relation_preimage_has_witness(eq_relation[A], s, x)
            let y: A satisfy {
                s.contains(y) and eq_relation[A](x, y)
            }
            x = y
            v.contains(x)
        }
        if v.contains(x) {
            eq_relation[A](x, x)
            relation_preimage_intro(eq_relation[A], s, x, x)
            u.contains(x)
        }
        u.contains(x) = v.contains(x)
    }
    set_ext(u, v)
}

/// Image under relation composition is iterated relational image.
theorem relation_image_compose[A, B, C](r: (A, B) -> Bool, s: (B, C) -> Bool, a: Set[A]) {
    relation_image(relation_compose(r, s), a) = relation_image(s, relation_image(r, a))
} by {
    let u = relation_image(relation_compose(r, s), a)
    let v = relation_image(s, relation_image(r, a))
    forall(z: C) {
        if u.contains(z) {
            relation_image_has_witness(relation_compose(r, s), a, z)
            let x: A satisfy {
                a.contains(x) and relation_compose(r, s, x, z)
            }
            relation_compose_has_witness(r, s, x, z)
            let y: B satisfy {
                r(x, y) and s(y, z)
            }
            relation_image_intro(r, a, x, y)
            relation_image(r, a).contains(y)
            relation_image_intro(s, relation_image(r, a), y, z)
            v.contains(z)
        }
        if v.contains(z) {
            relation_image_has_witness(s, relation_image(r, a), z)
            let y: B satisfy {
                relation_image(r, a).contains(y) and s(y, z)
            }
            relation_image_has_witness(r, a, y)
            let x: A satisfy {
                a.contains(x) and r(x, y)
            }
            relation_compose_intro(r, s, x, y, z)
            relation_compose(r, s, x, z)
            relation_image_intro(relation_compose(r, s), a, x, z)
            u.contains(z)
        }
        u.contains(z) = v.contains(z)
    }
    set_ext(u, v)
}

/// Preimage under relation composition is iterated relational preimage.
theorem relation_preimage_compose[A, B, C](r: (A, B) -> Bool, s: (B, C) -> Bool, c: Set[C]) {
    relation_preimage(relation_compose(r, s), c) = relation_preimage(r, relation_preimage(s, c))
} by {
    let u = relation_preimage(relation_compose(r, s), c)
    let v = relation_preimage(r, relation_preimage(s, c))
    forall(x: A) {
        if u.contains(x) {
            relation_preimage_has_witness(relation_compose(r, s), c, x)
            let z: C satisfy {
                c.contains(z) and relation_compose(r, s, x, z)
            }
            relation_compose_has_witness(r, s, x, z)
            let y: B satisfy {
                r(x, y) and s(y, z)
            }
            relation_preimage_intro(s, c, y, z)
            relation_preimage(s, c).contains(y)
            relation_preimage_intro(r, relation_preimage(s, c), x, y)
            v.contains(x)
        }
        if v.contains(x) {
            relation_preimage_has_witness(r, relation_preimage(s, c), x)
            let y: B satisfy {
                relation_preimage(s, c).contains(y) and r(x, y)
            }
            relation_preimage_has_witness(s, c, y)
            let z: C satisfy {
                c.contains(z) and s(y, z)
            }
            relation_compose_intro(r, s, x, y, z)
            relation_compose(r, s, x, z)
            relation_preimage_intro(relation_compose(r, s), c, x, z)
            u.contains(x)
        }
        u.contains(x) = v.contains(x)
    }
    set_ext(u, v)
}

/// Source restriction is contained in the original relation.
theorem relation_restrict_source_subset[A, B](r: (A, B) -> Bool, s: Set[A]) {
    relation_subset(relation_restrict_source(r, s), r)
} by {
    forall(x: A, y: B) {
        if relation_restrict_source(r, s, x, y) {
            r(x, y)
        }
    }
}

/// Target corestriction is contained in the original relation.
theorem relation_corestrict_target_subset[A, B](r: (A, B) -> Bool, t: Set[B]) {
    relation_subset(relation_corestrict_target(r, t), r)
} by {
    forall(x: A, y: B) {
        if relation_corestrict_target(r, t, x, y) {
            r(x, y)
        }
    }
}

/// Imaging through a source restriction is imaging from the intersection with that source set.
theorem relation_image_restrict_source[A, B](r: (A, B) -> Bool, s: Set[A], t: Set[A]) {
    relation_image(relation_restrict_source(r, s), t) = relation_image(r, s.intersection(t))
} by {
    let u = relation_image(relation_restrict_source(r, s), t)
    let v = relation_image(r, s.intersection(t))
    forall(y: B) {
        if u.contains(y) {
            relation_image_has_witness(relation_restrict_source(r, s), t, y)
            let x: A satisfy {
                t.contains(x) and relation_restrict_source(r, s, x, y)
            }
            s.contains(x)
            r(x, y)
            intersection_contains_intro(s, t, x)
            s.intersection(t).contains(x)
            relation_image_intro(r, s.intersection(t), x, y)
            v.contains(y)
        }
        if v.contains(y) {
            relation_image_has_witness(r, s.intersection(t), y)
            let x: A satisfy {
                s.intersection(t).contains(x) and r(x, y)
            }
            intersection_contains_eq(s, t, x)
            s.contains(x)
            t.contains(x)
            relation_restrict_source(r, s, x, y)
            relation_image_intro(relation_restrict_source(r, s), t, x, y)
            u.contains(y)
        }
        u.contains(y) = v.contains(y)
    }
    set_ext(u, v)
}

/// Preimaging through a target corestriction is preimaging from the intersection with that target set.
theorem relation_preimage_corestrict_target[A, B](r: (A, B) -> Bool, s: Set[B], t: Set[B]) {
    relation_preimage(relation_corestrict_target(r, s), t) = relation_preimage(r, s.intersection(t))
} by {
    let u = relation_preimage(relation_corestrict_target(r, s), t)
    let v = relation_preimage(r, s.intersection(t))
    forall(x: A) {
        if u.contains(x) {
            relation_preimage_has_witness(relation_corestrict_target(r, s), t, x)
            let y: B satisfy {
                t.contains(y) and relation_corestrict_target(r, s, x, y)
            }
            r(x, y)
            s.contains(y)
            intersection_contains_intro(s, t, y)
            s.intersection(t).contains(y)
            relation_preimage_intro(r, s.intersection(t), x, y)
            v.contains(x)
        }
        if v.contains(x) {
            relation_preimage_has_witness(r, s.intersection(t), x)
            let y: B satisfy {
                s.intersection(t).contains(y) and r(x, y)
            }
            intersection_contains_eq(s, t, y)
            s.contains(y)
            t.contains(y)
            relation_corestrict_target(r, s, x, y)
            relation_preimage_intro(relation_corestrict_target(r, s), t, x, y)
            u.contains(x)
        }
        u.contains(x) = v.contains(x)
    }
    set_ext(u, v)
}

/// Images through a target corestriction land in the target set.
theorem relation_image_corestrict_target_subset[A, B](r: (A, B) -> Bool, s: Set[A], t: Set[B]) {
    relation_image(relation_corestrict_target(r, t), s).subset(t)
} by {
    forall(y: B) {
        if relation_image(relation_corestrict_target(r, t), s).contains(y) {
            relation_image_has_witness(relation_corestrict_target(r, t), s, y)
            let x: A satisfy {
                s.contains(x) and relation_corestrict_target(r, t, x, y)
            }
            t.contains(y)
        }
    }
}

/// Preimages through a source restriction stay inside the source set.
theorem relation_preimage_restrict_source_subset[A, B](r: (A, B) -> Bool, s: Set[A], t: Set[B]) {
    relation_preimage(relation_restrict_source(r, s), t).subset(s)
} by {
    forall(x: A) {
        if relation_preimage(relation_restrict_source(r, s), t).contains(x) {
            relation_preimage_has_witness(relation_restrict_source(r, s), t, x)
            let y: B satisfy {
                t.contains(y) and relation_restrict_source(r, s, x, y)
            }
            s.contains(x)
        }
    }
}
