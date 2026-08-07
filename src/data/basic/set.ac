from data.basic.set_base import Set
from list import List
from list import filter_contains_and, filter_contained_by_and, filter_equivalent_to_and,
    range_contains_all_leq, range_does_not_contain_geq
from list import map_contains, map_contains_of_contains, map_length, injective_map_is_unique
from nat import Nat
from nat import add_imp_sub, lte_antisymm
from data.basic.functions import Inhabited, compose, function_extensionality, inverse_fn,
    inverse_fn_apply_of_surjective, is_injective_fn, is_surjective_fn, is_bijection_fn,
    bijection_fn_is_injective, bijection_fn_is_surjective, injective_fn_eq,
    surjective_fn_has_preimage
from data.basic.relation_basic import is_symmetric, is_transitive, is_equivalence, is_partial_equivalence,
    eq_relation, relation_converse, symmetric_flip, equivalence_flip, equivalence_self,
    equivalence_step, equivalence_imp_relation_converse_eq, partial_equivalence_flip,
    partial_equivalence_is_transitive, transitive_step
from data.basic.relation_transport import respects_equivalence

// Obvious sets
define constant_false[K](x: K) -> Bool {
    false
}
define negate_fun[K](f: K -> Bool, x: K) -> Bool {
    not f(x)
}
define singleton_fun[K](a: K, x: K) -> Bool {
    a = x
}

/// Creates a new function that returns true for the given item and delegates to the original function for all other inputs.
define functional_insert[K](f: K -> Bool, item: K, x: K) -> Bool {
    if x = item {
        true
    } else {
        f(x)
    }
}

/// Creates a new function that returns false for the given item and delegates to the original function for all other inputs.
define functional_remove[K](f: K -> Bool, item: K, x: K) -> Bool {
    if x = item {
        false
    } else {
        f(x)
    }
}

/// True if a boolean function represents a finite set.
/// A function satisfies the finite constraint if there exists a finite list containing all elements for which the function returns true.
define finite_constraint[K](contains: K -> Bool) -> Bool {
    exists(superset: List[K]) {
        forall(x: K) {
            contains(x) implies superset.contains(x)
        }
    }
}

theorem constant_false_satisfies_finite_constraint[K] {
    finite_constraint(constant_false[K])
}

theorem list_contains_satisfies_finite_constraint[K](ts: List[K]) {
    finite_constraint(ts.contains)
}

theorem functional_insert_satisfies_finite_constraint[K](f: K -> Bool, item: K) {
    finite_constraint(f) implies finite_constraint(functional_insert(f, item))
} by {
    if finite_constraint(f) {
        let superset: List[K] satisfy {
            forall(x: K) {
                f(x) implies superset.contains(x)
            }
        }
        let new_superset = List.cons(item, superset)
        forall(x: K) {
            if functional_insert(f, item, x) {
                if x = item {
                    new_superset.contains(x)
                } else {
                    f(x)
                    superset.contains(x)
                    new_superset.contains(x)
                }
            }
        }
        finite_constraint(functional_insert(f, item))
    }
}

theorem functional_remove_satisfies_finite_constraint[K](f: K -> Bool, item: K) {
    finite_constraint(f) implies finite_constraint(functional_remove(f, item))
} by {
    if finite_constraint(f) {
        let superset: List[K] satisfy {
            forall(x: K) {
                f(x) implies superset.contains(x)
            }
        }
        forall(x: K) {
            if functional_remove(f, item, x) {
                f(x)
                superset.contains(x)
            }
        }
        finite_constraint(functional_remove(f, item))
    }
}

theorem neg_of_neg_is_self[K](f: K -> Bool) {
    negate_fun(negate_fun(f)) = f
} by {
    forall(x: K) {
        // Trick that helps the prover, thanks @lacker
        if f(x) {
            negate_fun(negate_fun(f), x) = f(x)
        } else {
            negate_fun(negate_fun(f), x) = f(x)
        }
    }
}

/// Set basics
attributes Set[K] {
    let empty_set = Set[K].new(constant_false[K])
    /// The universal set containing all elements of type K.
    let universal_set = Set[K].new(negate_fun(constant_false[K]))
    /// Creates a set containing exactly one element.
    let singleton: K -> Set[K] = function(a: K) {
        Set[K].new(singleton_fun(a))
    }

    /// True if the set has no elements.
    define is_empty(self) -> Bool {
        forall(x: K) {
            not self.contains(x)
        }
    }

    /// The complement of this set.
    define c(self) -> Set[K] {
        Set[K].new(negate_fun(self.contains))
    }

    /// True if the set contains exactly one element.
    define is_singleton(self) -> Bool {
        exists(a: K) {
            self = Set[K].singleton(a)
        }
    }

    /// Removes an element from the set. If the item isn't present, this is a no-op.
    define remove(self, item: K) -> Set[K] {
        Set[K].new(functional_remove(self.contains, item))
    }

    /// Adds an element to the set. If the item is already present, this is a no-op.
    define insert(self, item: K) -> Set[K] {
        Set[K].new(functional_insert(self.contains, item))
    }

    /// True if the two sets have no elements in common.
    define is_disjoint(self, other: Set[K]) -> Bool {
        forall(x: K) {
            not (self.contains(x) and other.contains(x))
        }
    }

}

/// Membership in the empty set is always false.
theorem empty_set_contains_eq[K](x: K) {
    (Set[K].empty_set).contains(x) = false
}

theorem empty_set_is_empty[K] {
    (Set[K].empty_set).is_empty
} by {
    forall(x: K) {
        empty_set_contains_eq[K](x)
        not (Set[K].empty_set).contains(x)
    }
}

theorem singleton_set_is_singleton[K](a: K) {
    (Set[K].singleton(a)).is_singleton
} by {
    exists(b: K) {
        Set[K].singleton(a) = Set[K].singleton(b)
    }
}

/// Set extensionality: two sets are equal when they have the same elements.
theorem set_ext[K](a: Set[K], b: Set[K]) {
    (forall(x: K) { a.contains(x) = b.contains(x) }) implies a = b
} by {
    if forall(x: K) { a.contains(x) = b.contains(x) } {
        a.contains = b.contains
    }
}

/// Equal sets have equal membership predicates.
theorem set_eq_contains[K](a: Set[K], b: Set[K]) {
    a = b implies a.contains = b.contains
}

/// Equal sets have equal membership at every element.
theorem set_eq_contains_at[K](a: Set[K], b: Set[K], x: K) {
    a = b implies a.contains(x) = b.contains(x)
} by {
    if a = b {
        a.contains = b.contains
        a.contains(x) = b.contains(x)
    }
}

/// Equality of sets transports predicates on sets.
theorem set_eq_transport_predicate[K](p: Set[K] -> Bool, a: Set[K], b: Set[K]) {
    a = b and p(a) implies p(b)
}

/// Equality of sets transports predicates on sets in the reverse direction.
theorem set_eq_transport_predicate_rev[K](p: Set[K] -> Bool, a: Set[K], b: Set[K]) {
    a = b and p(b) implies p(a)
}

/// Equality of membership predicates determines equality of sets.
theorem set_eq_of_contains_eq[K](a: Set[K], b: Set[K]) {
    a.contains = b.contains implies a = b
} by {
    if a.contains = b.contains {
        forall(x: K) {
            a.contains(x) = b.contains(x)
        }
        set_ext(a, b)
    }
}

/// Pointwise equality of membership determines equality of sets.
theorem set_eq_of_contains_at_eq[K](a: Set[K], b: Set[K]) {
    (forall(x: K) { a.contains(x) = b.contains(x) }) implies a = b
} by {
    if forall(x: K) { a.contains(x) = b.contains(x) } {
        set_ext(a, b)
    }
}

/// A set with no elements is the empty set.
theorem set_eq_empty_of_is_empty[K](s: Set[K]) {
    s.is_empty implies s = Set[K].empty_set
} by {
    if s.is_empty {
        forall(x: K) {
            if s.contains(x) {
                false
            }
            empty_set_contains_eq[K](x)
            s.contains(x) = (Set[K].empty_set).contains(x)
        }
        set_ext(s, Set[K].empty_set)
    }
}

/// The characteristic predicate of the support of a relation.
define relation_support_fun[T](r: (T, T) -> Bool, x: T) -> Bool {
    exists(y: T) {
        r(x, y)
    }
}

/// The support of a relation, consisting of elements related to something.
define relation_support[T](r: (T, T) -> Bool) -> Set[T] {
    Set[T].new(relation_support_fun(r))
}

/// The characteristic predicate of the equivalence class of `a`.
define equivalence_class_fun[T](r: (T, T) -> Bool, a: T, x: T) -> Bool {
    r(a, x)
}

/// The equivalence class of `a` under the relation `r`.
define equivalence_class[T](r: (T, T) -> Bool, a: T) -> Set[T] {
    Set[T].new(equivalence_class_fun(r, a))
}

/// True if membership in `s` is preserved along the relation `r`.
define is_saturated_under[T](s: Set[T], r: (T, T) -> Bool) -> Bool {
    forall(x: T, y: T) {
        s.contains(x) and r(x, y) implies s.contains(y)
    }
}

/// Membership in the support is given by the support predicate.
theorem relation_support_contains_eq[T](r: (T, T) -> Bool, x: T) {
    relation_support(r).contains(x) = relation_support_fun(r, x)
}

/// A support element comes with a relation witness.
theorem relation_support_contains_witness[T](r: (T, T) -> Bool, x: T) {
    relation_support(r).contains(x) implies exists(y: T) {
        r(x, y)
    }
} by {
    if relation_support(r).contains(x) {
        relation_support_contains_eq(r, x)
        relation_support_fun(r, x)
        exists(y: T) {
            r(x, y)
        }
    }
}

/// Any related left endpoint lies in the support.
theorem relation_support_contains_of_related_left[T](r: (T, T) -> Bool, x: T, y: T) {
    r(x, y) implies relation_support(r).contains(x)
} by {
    if r(x, y) {
        exists(z: T) {
            z = y and r(x, z)
        }
        relation_support_fun(r, x)
        relation_support_contains_eq(r, x)
        relation_support(r).contains(x)
    }
}

/// Symmetry moves support membership across a related pair.
theorem relation_support_contains_of_related_right[T](r: (T, T) -> Bool, x: T, y: T) {
    is_symmetric(r) and r(x, y) implies relation_support(r).contains(y)
} by {
    if is_symmetric(r) and r(x, y) {
        symmetric_flip(r, x, y)
        r(y, x)
        relation_support_contains_of_related_left(r, y, x)
        relation_support(r).contains(y)
    }
}

/// Membership in an equivalence class is given by the underlying relation.
theorem equivalence_class_contains_eq[T](r: (T, T) -> Bool, a: T, x: T) {
    equivalence_class(r, a).contains(x) = r(a, x)
}

/// Any related element belongs to the equivalence class.
theorem equivalence_class_contains_of_related[T](r: (T, T) -> Bool, a: T, b: T) {
    r(a, b) implies equivalence_class(r, a).contains(b)
} by {
    if r(a, b) {
        equivalence_class_contains_eq(r, a, b)
        equivalence_class(r, a).contains(b)
    }
}

/// Equivalent elements have the same equivalence class.
theorem equivalence_class_membership_eq_of_equiv[T](r: (T, T) -> Bool, a: T, b: T, x: T) {
    is_equivalence(r) and r(a, b) implies
    equivalence_class(r, a).contains(x) = equivalence_class(r, b).contains(x)
} by {
    if is_equivalence(r) and r(a, b) {
        if equivalence_class(r, a).contains(x) {
            equivalence_class_contains_eq(r, a, x)
            equivalence_flip(r, a, b)
            r(b, a)
            equivalence_step(r, b, a, x)
            r(b, x)
            equivalence_class_contains_eq(r, b, x)
            equivalence_class(r, b).contains(x)
        }
        if equivalence_class(r, b).contains(x) {
            equivalence_class_contains_eq(r, b, x)
            equivalence_step(r, a, b, x)
            r(a, x)
            equivalence_class_contains_eq(r, a, x)
            equivalence_class(r, a).contains(x)
        }
        equivalence_class(r, a).contains(x) = equivalence_class(r, b).contains(x)
    }
}

/// The equivalence-class map respects the underlying equivalence relation.
theorem equivalence_class_respects_equivalence[T](r: (T, T) -> Bool) {
    is_equivalence(r) implies respects_equivalence(equivalence_class(r), r, eq_relation[Set[T]])
} by {
    if is_equivalence(r) {
        forall(a: T, b: T) {
            if r(a, b) {
                let u = equivalence_class(r, a)
                let v = equivalence_class(r, b)
                forall(x: T) {
                    equivalence_class_membership_eq_of_equiv(r, a, b, x)
                    u.contains(x) = v.contains(x)
                }
                set_ext(u, v)
                eq_relation[Set[T]](equivalence_class(r, a), equivalence_class(r, b))
            }
        }
    }
}

/// Equivalent elements have the same equivalence class.
theorem equivalence_class_eq_of_equiv[T](r: (T, T) -> Bool, a: T, b: T) {
    is_equivalence(r) and r(a, b) implies equivalence_class(r, a) = equivalence_class(r, b)
} by {
    if is_equivalence(r) and r(a, b) {
        equivalence_class_respects_equivalence(r)
        respects_equivalence(equivalence_class(r), r, eq_relation[Set[T]])
        eq_relation[Set[T]](equivalence_class(r, a), equivalence_class(r, b))
        equivalence_class(r, a) = equivalence_class(r, b)
    }
}

/// Equal equivalence classes force their representatives to be equivalent.
theorem equivalence_of_equivalence_class_eq[T](r: (T, T) -> Bool, a: T, b: T) {
    is_equivalence(r) and equivalence_class(r, a) = equivalence_class(r, b) implies r(a, b)
} by {
    if is_equivalence(r) and equivalence_class(r, a) = equivalence_class(r, b) {
        equivalence_self(r, a)
        equivalence_class_contains_eq(r, a, a)
        equivalence_class(r, a).contains(a)
        equivalence_class(r, b).contains(a)
        equivalence_class_contains_eq(r, b, a)
        r(b, a)
        relation_converse(r, a, b)
        equivalence_imp_relation_converse_eq(r)
        r(a, b)
    }
}

/// Every equivalence class is saturated under its equivalence relation.
theorem equivalence_class_is_saturated[T](r: (T, T) -> Bool, a: T) {
    is_equivalence(r) implies is_saturated_under(equivalence_class(r, a), r)
} by {
    if is_equivalence(r) {
        forall(x: T, y: T) {
            if equivalence_class(r, a).contains(x) and r(x, y) {
                equivalence_class_contains_eq(r, a, x)
                r(a, x)
                equivalence_step(r, a, x, y)
                r(a, y)
                equivalence_class_contains_eq(r, a, y)
                equivalence_class(r, a).contains(y)
            }
        }
    }
}

/// A saturated set containing `a` contains the whole equivalence class of `a`.
theorem saturated_contains_equivalence_class[T](s: Set[T], r: (T, T) -> Bool, a: T, x: T) {
    is_saturated_under(s, r) and s.contains(a) and equivalence_class(r, a).contains(x) implies s.contains(x)
} by {
    if is_saturated_under(s, r) and s.contains(a) and equivalence_class(r, a).contains(x) {
        is_saturated_under(s, r) = forall(y1: T, y2: T) {
            s.contains(y1) and r(y1, y2) implies s.contains(y2)
        }
        forall(y1: T, y2: T) {
            s.contains(y1) and r(y1, y2) implies s.contains(y2)
        }
        equivalence_class_contains_eq(r, a, x)
        r(a, x)
        s.contains(a) and r(a, x)
        s.contains(a) and r(a, x) implies s.contains(x)
        s.contains(x)
    }
}

/// A supported element is related to itself under a partial equivalence relation.
theorem partial_equivalence_self_of_support[T](r: (T, T) -> Bool, x: T) {
    is_partial_equivalence(r) and relation_support(r).contains(x) implies r(x, x)
} by {
    if is_partial_equivalence(r) and relation_support(r).contains(x) {
        relation_support_contains_witness(r, x)
        let y: T satisfy {
            r(x, y)
        }
        partial_equivalence_flip(r, x, y)
        r(y, x)
        partial_equivalence_is_transitive(r)
        is_transitive(r)
        transitive_step(r, x, y, x)
        r(x, x)
    }
}

/// Self-related elements lie in the support.
theorem partial_equivalence_support_contains_of_self[T](r: (T, T) -> Bool, x: T) {
    r(x, x) implies relation_support(r).contains(x)
} by {
    if r(x, x) {
        relation_support_contains_of_related_left(r, x, x)
        relation_support(r).contains(x)
    }
}

/// On a partial equivalence relation, support membership is equivalent to self-relatedness.
theorem partial_equivalence_support_iff_self[T](r: (T, T) -> Bool, x: T) {
    is_partial_equivalence(r) implies relation_support(r).contains(x) = r(x, x)
} by {
    if is_partial_equivalence(r) {
        if relation_support(r).contains(x) {
            partial_equivalence_self_of_support(r, x)
            r(x, x)
        }
        if r(x, x) {
            partial_equivalence_support_contains_of_self(r, x)
            relation_support(r).contains(x)
        }
        relation_support(r).contains(x) = r(x, x)
    }
}

/// Any related left endpoint lies in the support of a partial equivalence relation.
theorem partial_equivalence_related_imp_support_left[T](r: (T, T) -> Bool, a: T, b: T) {
    is_partial_equivalence(r) and r(a, b) implies relation_support(r).contains(a)
} by {
    if is_partial_equivalence(r) and r(a, b) {
        relation_support_contains_of_related_left(r, a, b)
        relation_support(r).contains(a)
    }
}

/// Any related right endpoint also lies in the support of a partial equivalence relation.
theorem partial_equivalence_related_imp_support_right[T](r: (T, T) -> Bool, a: T, b: T) {
    is_partial_equivalence(r) and r(a, b) implies relation_support(r).contains(b)
} by {
    if is_partial_equivalence(r) and r(a, b) {
        partial_equivalence_flip(r, a, b)
        r(b, a)
        relation_support_contains_of_related_left(r, b, a)
        relation_support(r).contains(b)
    }
}

/// The support of a partial equivalence relation is saturated.
theorem partial_equivalence_support_is_saturated[T](r: (T, T) -> Bool) {
    is_partial_equivalence(r) implies is_saturated_under(relation_support(r), r)
} by {
    if is_partial_equivalence(r) {
        forall(x: T, y: T) {
            if relation_support(r).contains(x) and r(x, y) {
                partial_equivalence_related_imp_support_right(r, x, y)
                relation_support(r).contains(y)
            }
        }
    }
}

/// A supported representative belongs to its own class.
theorem partial_equivalence_class_contains_self_of_support[T](r: (T, T) -> Bool, a: T) {
    is_partial_equivalence(r) and relation_support(r).contains(a) implies equivalence_class(r, a).contains(a)
} by {
    if is_partial_equivalence(r) and relation_support(r).contains(a) {
        partial_equivalence_self_of_support(r, a)
        r(a, a)
        equivalence_class_contains_eq(r, a, a)
        equivalence_class(r, a).contains(a)
    }
}

/// Related elements have the same class under a partial equivalence relation.
theorem partial_equivalence_class_membership_eq_of_related[T](r: (T, T) -> Bool, a: T, b: T, x: T) {
    is_partial_equivalence(r) and r(a, b) implies
    equivalence_class(r, a).contains(x) = equivalence_class(r, b).contains(x)
} by {
    if is_partial_equivalence(r) and r(a, b) {
        if equivalence_class(r, a).contains(x) {
            equivalence_class_contains_eq(r, a, x)
            partial_equivalence_flip(r, a, b)
            r(b, a)
            partial_equivalence_is_transitive(r)
            is_transitive(r)
            transitive_step(r, b, a, x)
            r(b, x)
            equivalence_class_contains_eq(r, b, x)
            equivalence_class(r, b).contains(x)
        }
        if equivalence_class(r, b).contains(x) {
            equivalence_class_contains_eq(r, b, x)
            partial_equivalence_is_transitive(r)
            is_transitive(r)
            transitive_step(r, a, b, x)
            r(a, x)
            equivalence_class_contains_eq(r, a, x)
            equivalence_class(r, a).contains(x)
        }
        equivalence_class(r, a).contains(x) = equivalence_class(r, b).contains(x)
    }
}

/// Related elements have equal classes under a partial equivalence relation.
theorem partial_equivalence_class_eq_of_related[T](r: (T, T) -> Bool, a: T, b: T) {
    is_partial_equivalence(r) and r(a, b) implies equivalence_class(r, a) = equivalence_class(r, b)
} by {
    if is_partial_equivalence(r) and r(a, b) {
        let u = equivalence_class(r, a)
        let v = equivalence_class(r, b)
        forall(x: T) {
            partial_equivalence_class_membership_eq_of_related(r, a, b, x)
            u.contains(x) = v.contains(x)
        }
        set_ext(u, v)
        equivalence_class(r, a) = equivalence_class(r, b)
    }
}

/// Equal classes recover relation membership once the left representative is supported.
theorem partial_equivalence_of_class_eq_and_support[T](r: (T, T) -> Bool, a: T, b: T) {
    is_partial_equivalence(r) and relation_support(r).contains(a) and
    equivalence_class(r, a) = equivalence_class(r, b) implies r(a, b)
} by {
    if is_partial_equivalence(r) and relation_support(r).contains(a) and
        equivalence_class(r, a) = equivalence_class(r, b) {
        partial_equivalence_class_contains_self_of_support(r, a)
        equivalence_class(r, a).contains(a)
        equivalence_class(r, b).contains(a)
        equivalence_class_contains_eq(r, b, a)
        r(b, a)
        partial_equivalence_flip(r, b, a)
        r(a, b)
    }
}

/// The class of a supported element is saturated under a partial equivalence relation.
theorem partial_equivalence_class_is_saturated[T](r: (T, T) -> Bool, a: T) {
    is_partial_equivalence(r) and relation_support(r).contains(a) implies
    is_saturated_under(equivalence_class(r, a), r)
} by {
    if is_partial_equivalence(r) and relation_support(r).contains(a) {
        forall(x: T, y: T) {
            if equivalence_class(r, a).contains(x) and r(x, y) {
                equivalence_class_contains_eq(r, a, x)
                r(a, x)
                partial_equivalence_is_transitive(r)
                is_transitive(r)
                transitive_step(r, a, x, y)
                r(a, y)
                equivalence_class_contains_eq(r, a, y)
                equivalence_class(r, a).contains(y)
            }
        }
    }
}

/// A saturated set containing a supported representative contains its whole class.
theorem saturated_contains_partial_equivalence_class[T](s: Set[T], r: (T, T) -> Bool, a: T, x: T) {
    is_partial_equivalence(r) and is_saturated_under(s, r) and s.contains(a) and
    equivalence_class(r, a).contains(x) implies s.contains(x)
} by {
    if is_partial_equivalence(r) and is_saturated_under(s, r) and s.contains(a) and
        equivalence_class(r, a).contains(x) {
        is_saturated_under(s, r) = forall(y1: T, y2: T) {
            s.contains(y1) and r(y1, y2) implies s.contains(y2)
        }
        forall(y1: T, y2: T) {
            s.contains(y1) and r(y1, y2) implies s.contains(y2)
        }
        equivalence_class_contains_eq(r, a, x)
        r(a, x)
        s.contains(a) and r(a, x)
        s.contains(x)
    }
}

/// Any class of a supported element is contained in the support.
theorem partial_equivalence_support_contains_class[T](r: (T, T) -> Bool, a: T, x: T) {
    is_partial_equivalence(r) and relation_support(r).contains(a) and
    equivalence_class(r, a).contains(x) implies relation_support(r).contains(x)
} by {
    if is_partial_equivalence(r) and relation_support(r).contains(a) and
        equivalence_class(r, a).contains(x) {
        partial_equivalence_support_is_saturated(r)
        is_saturated_under(relation_support(r), r)
        saturated_contains_partial_equivalence_class(relation_support(r), r, a, x)
        relation_support(r).contains(x)
    }
}

/// Any class element forces the representative into the support.
theorem partial_equivalence_class_contains_imp_support_left[T](r: (T, T) -> Bool, a: T, x: T) {
    is_partial_equivalence(r) and equivalence_class(r, a).contains(x) implies relation_support(r).contains(a)
} by {
    if is_partial_equivalence(r) and equivalence_class(r, a).contains(x) {
        equivalence_class_contains_eq(r, a, x)
        r(a, x)
        relation_support_contains_of_related_left(r, a, x)
        relation_support(r).contains(a)
    }
}

/// Any class element also lies in the support.
theorem partial_equivalence_class_contains_imp_support_right[T](r: (T, T) -> Bool, a: T, x: T) {
    is_partial_equivalence(r) and equivalence_class(r, a).contains(x) implies relation_support(r).contains(x)
} by {
    if is_partial_equivalence(r) and equivalence_class(r, a).contains(x) {
        equivalence_class_contains_eq(r, a, x)
        r(a, x)
        partial_equivalence_related_imp_support_right(r, a, x)
        relation_support(r).contains(x)
    }
}

/// A supported class is nonempty.
theorem partial_equivalence_class_nonempty_of_support[T](r: (T, T) -> Bool, a: T) {
    is_partial_equivalence(r) and relation_support(r).contains(a) implies exists(x: T) {
        equivalence_class(r, a).contains(x)
    }
} by {
    if is_partial_equivalence(r) and relation_support(r).contains(a) {
        partial_equivalence_class_contains_self_of_support(r, a)
        equivalence_class(r, a).contains(a)
        exists(x: T) {
            x = a and equivalence_class(r, a).contains(x)
        }
    }
}

/// A nonempty class forces its representative into the support.
theorem partial_equivalence_support_of_class_nonempty[T](r: (T, T) -> Bool, a: T) {
    is_partial_equivalence(r) and exists(x: T) {
        equivalence_class(r, a).contains(x)
    } implies relation_support(r).contains(a)
} by {
    if is_partial_equivalence(r) and exists(x: T) {
        equivalence_class(r, a).contains(x)
    } {
        let x: T satisfy {
            equivalence_class(r, a).contains(x)
        }
        partial_equivalence_class_contains_imp_support_left(r, a, x)
        relation_support(r).contains(a)
    }
}

/// For a partial equivalence relation, support membership is equivalent to class nonemptiness.
theorem partial_equivalence_class_nonempty_iff_support[T](r: (T, T) -> Bool, a: T) {
    is_partial_equivalence(r) implies relation_support(r).contains(a) = exists(x: T) {
        equivalence_class(r, a).contains(x)
    }
} by {
    if is_partial_equivalence(r) {
        if relation_support(r).contains(a) {
            partial_equivalence_class_nonempty_of_support(r, a)
            exists(x: T) {
                equivalence_class(r, a).contains(x)
            }
        }
        if exists(x: T) {
            equivalence_class(r, a).contains(x)
        } {
            partial_equivalence_support_of_class_nonempty(r, a)
            relation_support(r).contains(a)
        }
        relation_support(r).contains(a) = exists(x: T) {
            equivalence_class(r, a).contains(x)
        }
    }
}

/// Supported class equality is equivalent to relatedness.
theorem partial_equivalence_class_eq_iff_of_support[T](r: (T, T) -> Bool, a: T, b: T) {
    is_partial_equivalence(r) and relation_support(r).contains(a) implies
    equivalence_class(r, a) = equivalence_class(r, b) = r(a, b)
} by {
    if is_partial_equivalence(r) and relation_support(r).contains(a) {
        if equivalence_class(r, a) = equivalence_class(r, b) {
            partial_equivalence_of_class_eq_and_support(r, a, b)
            r(a, b)
        }
        if r(a, b) {
            partial_equivalence_class_eq_of_related(r, a, b)
            equivalence_class(r, a) = equivalence_class(r, b)
        }
        equivalence_class(r, a) = equivalence_class(r, b) = r(a, b)
    }
}

/// Supported class equality also forces the right representative into the support.
theorem partial_equivalence_class_eq_imp_support_right[T](r: (T, T) -> Bool, a: T, b: T) {
    is_partial_equivalence(r) and relation_support(r).contains(a) and
    equivalence_class(r, a) = equivalence_class(r, b) implies relation_support(r).contains(b)
} by {
    if is_partial_equivalence(r) and relation_support(r).contains(a) and
        equivalence_class(r, a) = equivalence_class(r, b) {
        partial_equivalence_of_class_eq_and_support(r, a, b)
        r(a, b)
        partial_equivalence_related_imp_support_right(r, a, b)
        relation_support(r).contains(b)
    }
}

theorem compl_of_compl_is_self[K](s: Set[K]) {
    s.c.c = s
} by {
    forall(x: K) {
        if s.contains(x) {
            not s.c.contains(x)
            s.c.c.contains(x)
        } else {
            s.c.contains(x)
            not s.c.c.contains(x)
        }
        s.c.c.contains(x) = s.contains(x)
    }

    set_ext(s.c.c, s)
}

theorem empty_set_compl_is_universal[K] {
    (Set[K].empty_set).c = Set[K].universal_set
} by {
    let e = Set[K].empty_set
    let u = Set[K].universal_set

    forall(x: K) {
        not e.contains(x)
        e.c.contains(x)
        not constant_false(x)
        negate_fun(constant_false[K], x)
        u.contains(x)
        e.c.contains(x) = u.contains(x)
    }

    set_ext(e.c, u)
}

theorem universal_set_compl_is_empty[K] {
    (Set[K].universal_set).c = Set[K].empty_set
} by {
    let e = Set[K].empty_set
    let u = Set[K].universal_set

    forall(x: K) {
        not constant_false(x)
        negate_fun(constant_false[K], x)
        u.contains(x)
        not u.c.contains(x)
        not e.contains(x)
        u.c.contains(x) = e.contains(x)
    }

    set_ext(u.c, e)
}

/// Membership in a singleton is equality with its unique element.
theorem singleton_contains_eq[K](a: K, x: K) {
    (Set[K].singleton(a)).contains(x) = (a = x)
}

theorem singleton_set_is_not_empty[K](a: K) {
    not (Set[K].singleton(a)).is_empty
} by {
    let s = Set[K].singleton(a)
    singleton_contains_eq(a, a)
    s.contains(a)
}

/// Membership in a complement is Boolean negation of membership.
theorem compl_contains_eq[K](s: Set[K], x: K) {
    s.c.contains(x) = not s.contains(x)
}

/// A set is disjoint from its complement.
theorem set_is_disjoint_compl[K](s: Set[K]) {
    s.is_disjoint(s.c)
} by {
    forall(x: K) {
        if s.contains(x) and s.c.contains(x) {
            compl_contains_eq(s, x)
            not s.contains(x)
            false
        }
        not (s.contains(x) and s.c.contains(x))
    }
}

/// Membership in the universal set is always true.
theorem universal_set_contains_eq[K](x: K) {
    (Set[K].universal_set).contains(x) = true
} by {
    not constant_false(x)
    negate_fun(constant_false[K], x)
}

/// A set containing every element is the universal set.
theorem set_eq_universal_of_forall_contains[K](s: Set[K]) {
    (forall(x: K) { s.contains(x) }) implies s = Set[K].universal_set
} by {
    if forall(x: K) { s.contains(x) } {
        forall(x: K) {
            universal_set_contains_eq[K](x)
            s.contains(x) = (Set[K].universal_set).contains(x)
        }
        set_ext(s, Set[K].universal_set)
    }
}

/// True if a natural number lies below a given bound.
define nat_lt_fun(bound: Nat, n: Nat) -> Bool {
    n < bound
}

/// The initial segment `{n : Nat | n < bound}`.
define nat_lt_set(bound: Nat) -> Set[Nat] {
    Set[Nat].new(nat_lt_fun(bound))
}

/// Membership in `nat_lt_set(bound)` is the inequality `n < bound`.
theorem nat_lt_set_contains_eq(bound: Nat, n: Nat) {
    nat_lt_set(bound).contains(n) = (n < bound)
}

/// True if a natural number is at least a given bound.
define at_least(bound: Nat, n: Nat) -> Bool {
    n >= bound
}

/// The final segment `{n : Nat | bound <= n}`.
define nat_from(bound: Nat) -> Set[Nat] {
    Set[Nat].new(at_least(bound))
}

/// Membership in a final segment is the corresponding lower-bound inequality.
theorem nat_from_contains_eq(bound: Nat, n: Nat) {
    nat_from(bound).contains(n) = (n >= bound)
} by {
    nat_from(bound).contains(n) = at_least(bound, n)
    at_least(bound, n) = (n >= bound)
}

/// True if all elements of the set are at least the given bound.
define set_lower_bound(s: Set[Nat], bound: Nat) -> Bool {
    forall(n: Nat) {
        s.contains(n) implies n >= bound
    }
}

// Subsets
attributes Set[K] {
    /// Set extensionality from pointwise equality of membership.
    let ext = set_ext[K]

    /// self ⊇ s
    define superset(self, s: Set[K]) -> Bool {
        s.subset(self)
    }
}

// Subset theorems
/// A set has lower bound `bound` exactly when it is contained in the final segment from `bound`.
theorem set_lower_bound_iff_subset_nat_from(s: Set[Nat], bound: Nat) {
    set_lower_bound(s, bound) = s.subset(nat_from(bound))
} by {
    if set_lower_bound(s, bound) {
        forall(n: Nat) {
            if s.contains(n) {
                set_lower_bound(s, bound) = forall(k: Nat) {
                    s.contains(k) implies k >= bound
                }
                s.contains(n) implies n >= bound
                n >= bound
                nat_from_contains_eq(bound, n)
                nat_from(bound).contains(n)
            }
        }
        s.subset(nat_from(bound))
    }
    if s.subset(nat_from(bound)) {
        forall(n: Nat) {
            if s.contains(n) {
                s.subset(nat_from(bound)) = forall(k: Nat) {
                    s.contains(k) implies nat_from(bound).contains(k)
                }
                s.contains(n) implies nat_from(bound).contains(n)
                nat_from(bound).contains(n)
                nat_from_contains_eq(bound, n)
                n >= bound
            }
        }
        set_lower_bound(s, bound)
    }
    set_lower_bound(s, bound) = s.subset(nat_from(bound))
}

/// A lower bound for a set is also a lower bound for every subset.
theorem set_lower_bound_of_subset(s: Set[Nat], t: Set[Nat], bound: Nat) {
    s.subset(t) and set_lower_bound(t, bound) implies set_lower_bound(s, bound)
} by {
    if s.subset(t) and set_lower_bound(t, bound) {
        forall(n: Nat) {
            if s.contains(n) {
                s.subset(t) = forall(k: Nat) {
                    s.contains(k) implies t.contains(k)
                }
                s.contains(n) implies t.contains(n)
                t.contains(n)
                set_lower_bound(t, bound) = forall(k: Nat) {
                    t.contains(k) implies k >= bound
                }
                t.contains(n) implies n >= bound
                n >= bound
            }
        }
        set_lower_bound(s, bound)
    }
}

/// The final segment from `bound` has lower bound `bound`.
theorem nat_from_has_lower_bound(bound: Nat) {
    set_lower_bound(nat_from(bound), bound)
} by {
    forall(n: Nat) {
        if nat_from(bound).contains(n) {
            nat_from_contains_eq(bound, n)
            n >= bound
        }
    }
    set_lower_bound(nat_from(bound), bound)
}

/// A set with a lower bound is contained in the corresponding final segment.
theorem nat_from_superset_of_bounded(s: Set[Nat], n: Nat) {
    set_lower_bound(s, n)
    implies
    nat_from(n).superset(s)
} by {
    if set_lower_bound(s, n) {
        set_lower_bound_iff_subset_nat_from(s, n)
        s.subset(nat_from(n))
        nat_from(n).superset(s)
    }
}

/// Subset membership is universal implication of membership.
theorem subset_contains_eq[K](a: Set[K], b: Set[K]) {
    a.subset(b) = forall(x: K) {
        a.contains(x) implies b.contains(x)
    }
}

/// Superset membership is universal implication of membership in the other direction.
theorem superset_contains_eq[K](a: Set[K], b: Set[K]) {
    a.superset(b) = forall(x: K) {
        b.contains(x) implies a.contains(x)
    }
}

/// A subset carries membership into its superset.
theorem subset_contains[K](a: Set[K], b: Set[K], x: K) {
    a.subset(b) and a.contains(x) implies b.contains(x)
} by {
    if a.subset(b) and a.contains(x) {
        subset_contains_eq(a, b)
        a.contains(x) implies b.contains(x)
        b.contains(x)
    }
}

/// A superset contains every element of the smaller set.
theorem superset_contains[K](a: Set[K], b: Set[K], x: K) {
    a.superset(b) and b.contains(x) implies a.contains(x)
} by {
    if a.superset(b) and b.contains(x) {
        superset_contains_eq(a, b)
        b.contains(x) implies a.contains(x)
        a.contains(x)
    }
}

/// Missing membership in a superset gives missing membership in any subset.
theorem not_contains_of_subset_not_contains[K](a: Set[K], b: Set[K], x: K) {
    a.subset(b) and not b.contains(x) implies not a.contains(x)
} by {
    if a.subset(b) and not b.contains(x) {
        if a.contains(x) {
            subset_contains(a, b, x)
            b.contains(x)
            false
        }
    }
}

/// Missing membership in a superset gives missing membership in any smaller set.
theorem not_contains_of_superset_not_contains[K](a: Set[K], b: Set[K], x: K) {
    a.superset(b) and not a.contains(x) implies not b.contains(x)
} by {
    if a.superset(b) and not a.contains(x) {
        if b.contains(x) {
            superset_contains(a, b, x)
            a.contains(x)
            false
        }
    }
}

theorem empty_set_is_always_subset[K](s: Set[K]) {
    (Set[K].empty_set).subset(s)
} by {
    forall(x: K) {
        if (Set[K].empty_set).contains(x) {
            false
        }
    }
}

theorem all_sets_subset_universal[K](s: Set[K]) {
    s.subset(Set[K].universal_set)
} by {
    let u = Set[K].universal_set
    forall(x: K) {
        not constant_false(x)
        negate_fun(constant_false[K], x)
        u.contains(x)
        s.contains(x) implies u.contains(x)
    }
}

theorem subset_refl[K](s: Set[K]) {
    s.subset(s)
} by {
    forall(x: K) {
        s.contains(x) implies s.contains(x)
    }
}

theorem subset_trans[K](a: Set[K], b: Set[K], c: Set[K]) {
    a.subset(b) and b.subset(c) implies a.subset(c)
} by {
    if a.subset(b) and b.subset(c) {
        forall(x: K) {
            if a.contains(x) {
                b.contains(x)
                c.contains(x)
            }
        }
    }
}

/// An initial segment is contained in its successor segment.
theorem nat_lt_set_subset_suc(bound: Nat) {
    nat_lt_set(bound).subset(nat_lt_set(bound.suc))
} by {
    forall(n: Nat) {
        if nat_lt_set(bound).contains(n) {
            nat_lt_set_contains_eq(bound, n)
            n < bound
            n < bound.suc
            nat_lt_set_contains_eq(bound.suc, n)
            nat_lt_set(bound.suc).contains(n)
        }
    }
}

theorem double_inclusion[K](a: Set[K], b: Set[K]) {
    a.subset(b) and b.subset(a) implies a = b
} by {
    forall(x: K) {
        b.contains(x) implies a.contains(x)
        a.contains(x) = b.contains(x)
    }
    set_ext(a, b)
}

/// Subset antisymmetry: two sets are equal when each is a subset of the other.
theorem subset_antisymm[K](a: Set[K], b: Set[K]) {
    a.subset(b) and b.subset(a) implies a = b
} by {
    double_inclusion(a, b)
}

/// Superset antisymmetry: two sets are equal when each is a superset of the other.
theorem superset_antisymm[K](a: Set[K], b: Set[K]) {
    a.superset(b) and b.superset(a) implies a = b
} by {
    if a.superset(b) and b.superset(a) {
        a.subset(b) and b.subset(a)
        double_inclusion(a, b)
    }
}

// "Standard" union and intersection of two sets
define elem_in_union[K](a: Set[K], b: Set[K], x: K) -> Bool {
    a.contains(x) or b.contains(x)
}

define elem_in_intersection[K](a: Set[K], b: Set[K], x: K) -> Bool {
    a.contains(x) and b.contains(x)
}

// Difference
define elem_in_difference[K](a: Set[K], b: Set[K], x: K) -> Bool {
    a.contains(x) and not b.contains(x)
}

attributes Set[K] {
    /// Subset antisymmetry from mutual inclusion.
    let antisymm = subset_antisymm[K]

    /// self ∪ s
    define union(self, s: Set[K]) -> Set[K] {
        Set.new(elem_in_union(self, s))
    }

    /// self ∩ s
    define intersection(self, s: Set[K]) -> Set[K] {
        Set[K].new(elem_in_intersection(self, s))
    }

    /// self \ s
    define difference(self, s: Set[K]) -> Set[K] {
        Set[K].new(elem_in_difference(self, s))
    }
}

/// True if `y` lies in the image of the set `s` under the function `f`.
define image_contains[T, U](f: T -> U, s: Set[T], y: U) -> Bool {
    exists(x: T) {
        s.contains(x) and y = f(x)
    }
}

/// The image of the set `s` under the function `f`.
define set_image[T, U](s: Set[T], f: T -> U) -> Set[U] {
    Set[U].new(image_contains(f, s))
}

/// Membership in the image set is equivalent to the image predicate.
theorem set_image_contains_eq[T, U](s: Set[T], f: T -> U, y: U) {
    set_image(s, f).contains(y) = image_contains(f, s, y)
}

/// True if `x` lies in the preimage of the set `s` under the function `f`.
define preimage_contains[T, U](f: T -> U, s: Set[U], x: T) -> Bool {
    s.contains(f(x))
}

/// The preimage of the set `s` under the function `f`.
define set_preimage[T, U](f: T -> U, s: Set[U]) -> Set[T] {
    Set[T].new(preimage_contains(f, s))
}

/// Membership in the preimage set is equivalent to the preimage predicate.
theorem set_preimage_contains_eq[T, U](f: T -> U, s: Set[U], x: T) {
    set_preimage(f, s).contains(x) = preimage_contains(f, s, x)
}

/// Membership in the preimage of `s` under `f` is equivalent to membership of `f(x)` in `s`.
theorem set_preimage_contains_image[T, U](f: T -> U, s: Set[U], x: T) {
    set_preimage(f, s).contains(x) = s.contains(f(x))
} by {
    set_preimage_contains_eq(f, s, x)
    preimage_contains(f, s, x) = s.contains(f(x))
}

/// The preimage of the empty set is empty.
theorem set_preimage_empty[T, U](f: T -> U) {
    set_preimage(f, Set[U].empty_set) = Set[T].empty_set
} by {
    let s = set_preimage(f, Set[U].empty_set)
    let e = Set[T].empty_set
    forall(x: T) {
        set_preimage_contains_eq(f, Set[U].empty_set, x)
        empty_set_contains_eq[U](f(x))
        empty_set_contains_eq[T](x)
        s.contains(x) = e.contains(x)
    }
    set_ext(s, e)
}

/// The preimage of the universal set is universal.
theorem set_preimage_universal[T, U](f: T -> U) {
    set_preimage(f, Set[U].universal_set) = Set[T].universal_set
} by {
    let s = set_preimage(f, Set[U].universal_set)
    let u = Set[T].universal_set
    forall(x: T) {
        set_preimage_contains_eq(f, Set[U].universal_set, x)
        universal_set_contains_eq[U](f(x))
        universal_set_contains_eq[T](x)
        s.contains(x) = u.contains(x)
    }
    set_ext(s, u)
}

/// Preimage is monotone with respect to subset.
theorem set_preimage_monotone[T, U](f: T -> U, a: Set[U], b: Set[U]) {
    a.subset(b) implies set_preimage(f, a).subset(set_preimage(f, b))
} by {
    if a.subset(b) {
        forall(x: T) {
            if set_preimage(f, a).contains(x) {
                set_preimage_contains_eq(f, a, x)
                a.contains(f(x))
                b.contains(f(x))
                set_preimage_contains_eq(f, b, x)
                set_preimage(f, b).contains(x)
            }
        }
    }
}

/// Preimage commutes with complement.
theorem set_preimage_compl[T, U](f: T -> U, s: Set[U]) {
    set_preimage(f, s.c) = set_preimage(f, s).c
} by {
    let u = set_preimage(f, s.c)
    let v = set_preimage(f, s).c
    forall(x: T) {
        if u.contains(x) {
            set_preimage_contains_eq(f, s.c, x)
            compl_contains_eq(s, f(x))
            not s.contains(f(x))
            set_preimage_contains_eq(f, s, x)
            not set_preimage(f, s).contains(x)
            compl_contains_eq(set_preimage(f, s), x)
            v.contains(x)
        }
        if v.contains(x) {
            compl_contains_eq(set_preimage(f, s), x)
            not set_preimage(f, s).contains(x)
            set_preimage_contains_eq(f, s, x)
            not s.contains(f(x))
            compl_contains_eq(s, f(x))
            set_preimage_contains_eq(f, s.c, x)
            u.contains(x)
        }
        u.contains(x) = v.contains(x)
    }
    set_ext(u, v)
}

/// Preimage under a composition is iterated preimage.
theorem set_preimage_compose[A, B, C](f: B -> C, g: A -> B, s: Set[C]) {
    set_preimage(compose(f, g), s) = set_preimage(g, set_preimage(f, s))
} by {
    let u = set_preimage(compose(f, g), s)
    let v = set_preimage(g, set_preimage(f, s))
    forall(x: A) {
        set_preimage_contains_eq(compose(f, g), s, x)
        compose(f, g, x) = f(g(x))
        set_preimage_contains_eq(f, s, g(x))
        set_preimage_contains_eq(g, set_preimage(f, s), x)
        u.contains(x) = v.contains(x)
    }
    set_ext(u, v)
}

/// An element of a set maps to an element of its image.
theorem maps_into_set_image[T, U](s: Set[T], f: T -> U, x: T) {
    s.contains(x) implies set_image(s, f).contains(f(x))
} by {
    if s.contains(x) {
        exists(z: T) {
            z = x and s.contains(z) and f(x) = f(z)
        }
        image_contains(f, s, f(x))
    }
}

/// Membership in an image yields a witness in the source set.
theorem set_image_contains_witness[T, U](s: Set[T], f: T -> U, y: U) {
    set_image(s, f).contains(y) implies exists(x: T) {
        s.contains(x) and y = f(x)
    }
} by {
    if set_image(s, f).contains(y) {
        set_image_contains_eq(s, f, y)
        image_contains(f, s, y)
    }
}

/// Membership in a bijective image gives membership of the chosen inverse.
theorem set_image_contains_inverse_of_bijection[T: Inhabited, U](s: Set[T], f: T -> U, y: U) {
    is_bijection_fn(f) and set_image(s, f).contains(y) implies s.contains(inverse_fn(f, y))
} by {
    if is_bijection_fn(f) and set_image(s, f).contains(y) {
        set_image_contains_witness(s, f, y)
        let x: T satisfy {
            s.contains(x) and y = f(x)
        }
        bijection_fn_is_surjective(f)
        is_surjective_fn(f)
        inverse_fn_apply_of_surjective(f, y)
        f(inverse_fn(f, y)) = y
        y = f(x)
        f(inverse_fn(f, y)) = f(x)
        bijection_fn_is_injective(f)
        is_injective_fn(f)
        injective_fn_eq(f, inverse_fn(f, y), x)
        inverse_fn(f, y) = x
        s.contains(inverse_fn(f, y))
    }
}

/// Membership of the chosen inverse gives membership in a surjective image.
theorem set_inverse_contains_imp_image_contains[T: Inhabited, U](s: Set[T], f: T -> U, y: U) {
    is_surjective_fn(f) and s.contains(inverse_fn(f, y)) implies set_image(s, f).contains(y)
} by {
    if is_surjective_fn(f) and s.contains(inverse_fn(f, y)) {
        inverse_fn_apply_of_surjective(f, y)
        f(inverse_fn(f, y)) = y
        maps_into_set_image(s, f, inverse_fn(f, y))
        set_image(s, f).contains(f(inverse_fn(f, y)))
        set_image(s, f).contains(y)
    }
}

/// Membership in a bijective image is membership of the chosen inverse.
theorem set_image_contains_iff_inverse_of_bijection[T: Inhabited, U](s: Set[T], f: T -> U, y: U) {
    is_bijection_fn(f) implies set_image(s, f).contains(y) = s.contains(inverse_fn(f, y))
} by {
    if is_bijection_fn(f) {
        if set_image(s, f).contains(y) {
            set_image_contains_inverse_of_bijection(s, f, y)
            s.contains(inverse_fn(f, y))
        }
        if s.contains(inverse_fn(f, y)) {
            bijection_fn_is_surjective(f)
            is_surjective_fn(f)
            set_inverse_contains_imp_image_contains(s, f, y)
            set_image(s, f).contains(y)
        }
        set_image(s, f).contains(y) = s.contains(inverse_fn(f, y))
    }
}

/// The image of the empty set is empty.
theorem set_image_empty[T, U](f: T -> U) {
    set_image(Set[T].empty_set, f) = Set[U].empty_set
} by {
    let s = set_image(Set[T].empty_set, f)
    let e = Set[U].empty_set
    forall(y: U) {
        if s.contains(y) {
            let x: T satisfy {
                Set[T].empty_set.contains(x) and y = f(x)
            }
            empty_set_contains_eq[T](x)
            false
        }
        empty_set_contains_eq[U](y)
        s.contains(y) = e.contains(y)
    }
    set_ext(s, e)
}

/// The image of a singleton is the singleton of the image.
theorem set_image_singleton[T, U](a: T, f: T -> U) {
    set_image(Set[T].singleton(a), f) = Set[U].singleton(f(a))
} by {
    let s = set_image(Set[T].singleton(a), f)
    let t = Set[U].singleton(f(a))
    forall(y: U) {
        if s.contains(y) {
            let x: T satisfy {
                Set[T].singleton(a).contains(x) and y = f(x)
            }
            singleton_contains_eq[T](a, x)
            x = a
            y = f(a)
            singleton_contains_eq[U](f(a), y)
            t.contains(y)
        }
        if t.contains(y) {
            singleton_contains_eq[U](f(a), y)
            y = f(a)
            singleton_contains_eq[T](a, a)
            Set[T].singleton(a).contains(a)
            maps_into_set_image(Set[T].singleton(a), f, a)
            s.contains(f(a))
            s.contains(y)
        }
        s.contains(y) = t.contains(y)
    }
    set_ext(s, t)
}

/// Image is monotone with respect to subset.
theorem set_image_monotone[T, U](a: Set[T], b: Set[T], f: T -> U) {
    a.subset(b) implies set_image(a, f).subset(set_image(b, f))
} by {
    if a.subset(b) {
        forall(y: U) {
            if set_image(a, f).contains(y) {
                let x: T satisfy {
                    a.contains(x) and y = f(x)
                }
                b.contains(x)
                exists(z: T) {
                    z = x and b.contains(z) and y = f(z)
                }
                image_contains(f, b, y)
            }
        }
    }
}

/// Image under a composition is iterated image.
theorem set_image_compose[A, B, C](s: Set[A], g: A -> B, f: B -> C) {
    set_image(set_image(s, g), f) = set_image(s, compose(f, g))
} by {
    let u = set_image(set_image(s, g), f)
    let v = set_image(s, compose(f, g))
    forall(y: C) {
        if u.contains(y) {
            set_image_contains_witness(set_image(s, g), f, y)
            let z: B satisfy {
                set_image(s, g).contains(z) and y = f(z)
            }
            set_image_contains_witness(s, g, z)
            let x: A satisfy {
                s.contains(x) and z = g(x)
            }
            z = g(x)
            y = f(g(x))
            compose(f, g, x) = f(g(x))
            y = compose(f, g, x)
            image_contains(compose(f, g), s, y)
            v.contains(y)
        }
        if v.contains(y) {
            set_image_contains_witness(s, compose(f, g), y)
            let x: A satisfy {
                s.contains(x) and y = compose(f, g, x)
            }
            compose(f, g, x) = f(g(x))
            y = f(g(x))
            maps_into_set_image(s, g, x)
            set_image(s, g).contains(g(x))
            maps_into_set_image(set_image(s, g), f, g(x))
            u.contains(f(g(x)))
            u.contains(y)
        }
        u.contains(y) = v.contains(y)
    }
    set_ext(u, v)
}

/// Every set is contained in the preimage of its image.
theorem subset_set_preimage_image[T, U](s: Set[T], f: T -> U) {
    s.subset(set_preimage(f, set_image(s, f)))
} by {
    forall(x: T) {
        if s.contains(x) {
            maps_into_set_image(s, f, x)
            set_image(s, f).contains(f(x))
            set_preimage_contains_eq(f, set_image(s, f), x)
            set_preimage(f, set_image(s, f)).contains(x)
        }
    }
}

/// Under injectivity, taking preimage after image recovers the original set.
theorem set_preimage_image_of_injective[T, U](s: Set[T], f: T -> U) {
    is_injective_fn(f) implies set_preimage(f, set_image(s, f)) = s
} by {
    let u = set_preimage(f, set_image(s, f))
    let v = s
    if is_injective_fn(f) {
        subset_set_preimage_image(s, f)
        s.subset(u)
        forall(x: T) {
            if u.contains(x) {
                set_preimage_contains_eq(f, set_image(s, f), x)
                set_image(s, f).contains(f(x))
                set_image_contains_witness(s, f, f(x))
                let x2: T satisfy {
                    s.contains(x2) and f(x) = f(x2)
                }
                injective_fn_eq(f, x, x2)
                x = x2
                s.contains(x)
            }
        }
        u.subset(v)
        double_inclusion(u, v)
        u = v
    }
}

/// The image of a preimage is contained in the target set.
theorem set_image_preimage_subset[T, U](s: Set[U], f: T -> U) {
    set_image(set_preimage(f, s), f).subset(s)
} by {
    forall(y: U) {
        if set_image(set_preimage(f, s), f).contains(y) {
            set_image_contains_witness(set_preimage(f, s), f, y)
            let x: T satisfy {
                set_preimage(f, s).contains(x) and y = f(x)
            }
            set_preimage_contains_eq(f, s, x)
            s.contains(f(x))
            s.contains(y)
        }
    }
}

/// Image-subset is equivalent to source-subset of the preimage.
theorem set_image_subset_iff_subset_preimage[T, U](s: Set[T], t: Set[U], f: T -> U) {
    set_image(s, f).subset(t) = s.subset(set_preimage(f, t))
} by {
    if set_image(s, f).subset(t) {
        forall(x: T) {
            if s.contains(x) {
                maps_into_set_image(s, f, x)
                set_image(s, f).contains(f(x))
                t.contains(f(x))
                set_preimage_contains_eq(f, t, x)
                set_preimage(f, t).contains(x)
            }
        }
        s.subset(set_preimage(f, t))
    }
    if s.subset(set_preimage(f, t)) {
        forall(y: U) {
            if set_image(s, f).contains(y) {
                set_image_contains_witness(s, f, y)
                let x: T satisfy {
                    s.contains(x) and y = f(x)
                }
                set_preimage(f, t).contains(x)
                set_preimage_contains_eq(f, t, x)
                t.contains(f(x))
                t.contains(y)
            }
        }
        set_image(s, f).subset(t)
    }
    set_image(s, f).subset(t) = s.subset(set_preimage(f, t))
}

/// A source subset of a preimage gives an image subset.
theorem set_image_subset_of_subset_preimage[T, U](s: Set[T], t: Set[U], f: T -> U) {
    s.subset(set_preimage(f, t)) implies set_image(s, f).subset(t)
} by {
    if s.subset(set_preimage(f, t)) {
        set_image_subset_iff_subset_preimage(s, t, f)
        set_image(s, f).subset(t)
    }
}

/// An image subset gives a source subset of the preimage.
theorem subset_preimage_of_set_image_subset[T, U](s: Set[T], t: Set[U], f: T -> U) {
    set_image(s, f).subset(t) implies s.subset(set_preimage(f, t))
} by {
    if set_image(s, f).subset(t) {
        set_image_subset_iff_subset_preimage(s, t, f)
        s.subset(set_preimage(f, t))
    }
}

/// True if a map between set families preserves inclusion.
define is_subset_monotone_map[T, U](phi: Set[T] -> Set[U]) -> Bool {
    forall(a: Set[T], b: Set[T]) {
        a.subset(b) implies phi(a).subset(phi(b))
    }
}

/// True if a self-map on sets contains every set in its image.
define is_set_extensive_map[T](phi: Set[T] -> Set[T]) -> Bool {
    forall(s: Set[T]) {
        s.subset(phi(s))
    }
}

/// True if a self-map on sets is contained in every set it acts on.
define is_set_reductive_map[T](phi: Set[T] -> Set[T]) -> Bool {
    forall(s: Set[T]) {
        phi(s).subset(s)
    }
}

/// True if a self-map on sets is unchanged after a second application.
define is_set_idempotent_map[T](phi: Set[T] -> Set[T]) -> Bool {
    forall(s: Set[T]) {
        phi(phi(s)) = phi(s)
    }
}

/// True if a self-map on sets is a closure operator for inclusion.
define is_set_closure_operator[T](close: Set[T] -> Set[T]) -> Bool {
    is_subset_monotone_map(close) and is_set_extensive_map(close) and
    is_set_idempotent_map(close)
}

/// True if a self-map on sets is a kernel operator for inclusion.
define is_set_kernel_operator[T](kernel: Set[T] -> Set[T]) -> Bool {
    is_subset_monotone_map(kernel) and is_set_reductive_map(kernel) and
    is_set_idempotent_map(kernel)
}

/// A set closure operator preserves inclusion.
theorem set_closure_operator_is_monotone[T](close: Set[T] -> Set[T]) {
    is_set_closure_operator(close) implies is_subset_monotone_map(close)
} by {
    if is_set_closure_operator(close) {
        is_set_closure_operator(close) =
            (is_subset_monotone_map(close) and is_set_extensive_map(close) and
            is_set_idempotent_map(close))
        is_subset_monotone_map(close)
    }
}

/// A set closure operator contains each set.
theorem set_closure_operator_is_extensive[T](close: Set[T] -> Set[T]) {
    is_set_closure_operator(close) implies is_set_extensive_map(close)
} by {
    if is_set_closure_operator(close) {
        is_set_closure_operator(close) =
            (is_subset_monotone_map(close) and is_set_extensive_map(close) and
            is_set_idempotent_map(close))
        is_set_extensive_map(close)
    }
}

/// A set closure operator is unchanged after two applications.
theorem set_closure_operator_is_idempotent[T](close: Set[T] -> Set[T]) {
    is_set_closure_operator(close) implies is_set_idempotent_map(close)
} by {
    if is_set_closure_operator(close) {
        is_set_closure_operator(close) =
            (is_subset_monotone_map(close) and is_set_extensive_map(close) and
            is_set_idempotent_map(close))
        is_set_idempotent_map(close)
    }
}

/// A set closure operator contains each set at a chosen set.
theorem set_closure_operator_extensive_at[T](close: Set[T] -> Set[T], s: Set[T]) {
    is_set_closure_operator(close) implies s.subset(close(s))
} by {
    if is_set_closure_operator(close) {
        set_closure_operator_is_extensive(close)
        is_set_extensive_map(close)
        is_set_extensive_map(close) = forall(t: Set[T]) {
            t.subset(close(t))
        }
        s.subset(close(s))
    }
}

/// A set closure operator is unchanged after two applications at a chosen set.
theorem set_closure_operator_idempotent_at[T](close: Set[T] -> Set[T], s: Set[T]) {
    is_set_closure_operator(close) implies close(close(s)) = close(s)
} by {
    if is_set_closure_operator(close) {
        set_closure_operator_is_idempotent(close)
        is_set_idempotent_map(close)
        is_set_idempotent_map(close) = forall(t: Set[T]) {
            close(close(t)) = close(t)
        }
        close(close(s)) = close(s)
    }
}

/// The value of a set closure operator is fixed by that operator.
theorem set_closure_operator_image_fixed[T](close: Set[T] -> Set[T], s: Set[T]) {
    is_set_closure_operator(close) implies close(close(s)) = close(s)
} by {
    if is_set_closure_operator(close) {
        set_closure_operator_idempotent_at(close, s)
        close(close(s)) = close(s)
    }
}

/// A closed set above a set also contains its closure.
theorem set_closure_operator_subset_fixed_of_subset[T](
    close: Set[T] -> Set[T],
    s: Set[T],
    c: Set[T]
) {
    is_set_closure_operator(close) and close(c) = c and s.subset(c) implies close(s).subset(c)
} by {
    if is_set_closure_operator(close) and close(c) = c and s.subset(c) {
        set_closure_operator_is_monotone(close)
        is_subset_monotone_map(close)
        is_subset_monotone_map(close) = forall(a: Set[T], b: Set[T]) {
            a.subset(b) implies close(a).subset(close(b))
        }
        close(s).subset(close(c))
        close(s).subset(c)
    }
}

/// Below a closed set, comparing the closure is the same as comparing the set.
theorem set_closure_operator_subset_fixed_iff_subset[T](
    close: Set[T] -> Set[T],
    s: Set[T],
    c: Set[T]
) {
    is_set_closure_operator(close) and close(c) = c implies (close(s).subset(c) = s.subset(c))
} by {
    if is_set_closure_operator(close) and close(c) = c {
        if close(s).subset(c) {
            set_closure_operator_extensive_at(close, s)
            s.subset(close(s))
            subset_trans(s, close(s), c)
            s.subset(c)
        }
        if s.subset(c) {
            set_closure_operator_subset_fixed_of_subset(close, s, c)
            close(s).subset(c)
        }
        close(s).subset(c) = s.subset(c)
    }
}

/// A set kernel operator preserves inclusion.
theorem set_kernel_operator_is_monotone[T](kernel: Set[T] -> Set[T]) {
    is_set_kernel_operator(kernel) implies is_subset_monotone_map(kernel)
} by {
    if is_set_kernel_operator(kernel) {
        is_set_kernel_operator(kernel) =
            (is_subset_monotone_map(kernel) and is_set_reductive_map(kernel) and
            is_set_idempotent_map(kernel))
        is_subset_monotone_map(kernel)
    }
}

/// A set kernel operator is contained in each set.
theorem set_kernel_operator_is_reductive[T](kernel: Set[T] -> Set[T]) {
    is_set_kernel_operator(kernel) implies is_set_reductive_map(kernel)
} by {
    if is_set_kernel_operator(kernel) {
        is_set_kernel_operator(kernel) =
            (is_subset_monotone_map(kernel) and is_set_reductive_map(kernel) and
            is_set_idempotent_map(kernel))
        is_set_reductive_map(kernel)
    }
}

/// A set kernel operator is unchanged after two applications.
theorem set_kernel_operator_is_idempotent[T](kernel: Set[T] -> Set[T]) {
    is_set_kernel_operator(kernel) implies is_set_idempotent_map(kernel)
} by {
    if is_set_kernel_operator(kernel) {
        is_set_kernel_operator(kernel) =
            (is_subset_monotone_map(kernel) and is_set_reductive_map(kernel) and
            is_set_idempotent_map(kernel))
        is_set_idempotent_map(kernel)
    }
}

/// A set kernel operator is contained in each set at a chosen set.
theorem set_kernel_operator_reductive_at[T](kernel: Set[T] -> Set[T], s: Set[T]) {
    is_set_kernel_operator(kernel) implies kernel(s).subset(s)
} by {
    if is_set_kernel_operator(kernel) {
        set_kernel_operator_is_reductive(kernel)
        is_set_reductive_map(kernel)
        is_set_reductive_map(kernel) = forall(t: Set[T]) {
            kernel(t).subset(t)
        }
        kernel(s).subset(s)
    }
}

/// A set kernel operator is unchanged after two applications at a chosen set.
theorem set_kernel_operator_idempotent_at[T](kernel: Set[T] -> Set[T], s: Set[T]) {
    is_set_kernel_operator(kernel) implies kernel(kernel(s)) = kernel(s)
} by {
    if is_set_kernel_operator(kernel) {
        set_kernel_operator_is_idempotent(kernel)
        is_set_idempotent_map(kernel)
        is_set_idempotent_map(kernel) = forall(t: Set[T]) {
            kernel(kernel(t)) = kernel(t)
        }
        kernel(kernel(s)) = kernel(s)
    }
}

/// The value of a set kernel operator is fixed by that operator.
theorem set_kernel_operator_image_fixed[T](kernel: Set[T] -> Set[T], s: Set[T]) {
    is_set_kernel_operator(kernel) implies kernel(kernel(s)) = kernel(s)
} by {
    if is_set_kernel_operator(kernel) {
        set_kernel_operator_idempotent_at(kernel, s)
        kernel(kernel(s)) = kernel(s)
    }
}

/// A kernel-stable set below a set is also contained in its kernel.
theorem set_fixed_subset_kernel_operator_of_subset[T](
    kernel: Set[T] -> Set[T],
    k: Set[T],
    s: Set[T]
) {
    is_set_kernel_operator(kernel) and kernel(k) = k and k.subset(s) implies k.subset(kernel(s))
} by {
    if is_set_kernel_operator(kernel) and kernel(k) = k and k.subset(s) {
        set_kernel_operator_is_monotone(kernel)
        is_subset_monotone_map(kernel)
        is_subset_monotone_map(kernel) = forall(a: Set[T], b: Set[T]) {
            a.subset(b) implies kernel(a).subset(kernel(b))
        }
        kernel(k).subset(kernel(s))
        k.subset(kernel(s))
    }
}

/// Above a kernel-stable set, comparing with the kernel is the same as comparing with the set.
theorem set_fixed_subset_kernel_operator_iff_subset[T](
    kernel: Set[T] -> Set[T],
    k: Set[T],
    s: Set[T]
) {
    is_set_kernel_operator(kernel) and kernel(k) = k implies (k.subset(kernel(s)) = k.subset(s))
} by {
    if is_set_kernel_operator(kernel) and kernel(k) = k {
        if k.subset(kernel(s)) {
            set_kernel_operator_reductive_at(kernel, s)
            kernel(s).subset(s)
            subset_trans(k, kernel(s), s)
            k.subset(s)
        }
        if k.subset(s) {
            set_fixed_subset_kernel_operator_of_subset(kernel, k, s)
            k.subset(kernel(s))
        }
        k.subset(kernel(s)) = k.subset(s)
    }
}

/// True if two set maps are adjoint through inclusion.
define is_set_galois_connection[T, U](
    lower: Set[T] -> Set[U],
    upper: Set[U] -> Set[T]
) -> Bool {
    forall(s: Set[T], t: Set[U]) {
        lower(s).subset(t) = s.subset(upper(t))
    }
}

/// The direct image map on subsets.
define set_image_map[T, U](f: T -> U, s: Set[T]) -> Set[U] {
    set_image(s, f)
}

/// The inverse image map on subsets.
define set_preimage_map[T, U](f: T -> U, s: Set[U]) -> Set[T] {
    set_preimage(f, s)
}

/// The direct image map is ordinary set image.
theorem set_image_map_at[T, U](f: T -> U, s: Set[T]) {
    set_image_map(f, s) = set_image(s, f)
}

/// The inverse image map is ordinary set preimage.
theorem set_preimage_map_at[T, U](f: T -> U, s: Set[U]) {
    set_preimage_map(f, s) = set_preimage(f, s)
}

/// A set Galois connection is exactly its inclusion-adjunction law.
theorem set_galois_connection_at[T, U](
    lower: Set[T] -> Set[U],
    upper: Set[U] -> Set[T],
    s: Set[T],
    t: Set[U]
) {
    is_set_galois_connection(lower, upper) implies
    (lower(s).subset(t) = s.subset(upper(t)))
} by {
    if is_set_galois_connection(lower, upper) {
        is_set_galois_connection(lower, upper) = forall(a: Set[T], b: Set[U]) {
            lower(a).subset(b) = a.subset(upper(b))
        }
        lower(s).subset(t) = s.subset(upper(t))
    }
}

/// The image and preimage maps of a function form a set Galois connection.
theorem set_image_preimage_is_set_galois_connection[T, U](f: T -> U) {
    is_set_galois_connection(set_image_map(f), set_preimage_map(f))
} by {
    forall(s: Set[T], t: Set[U]) {
        set_image_map_at(f, s)
        set_preimage_map_at(f, t)
        set_image_subset_iff_subset_preimage(s, t, f)
        set_image_map(f, s).subset(t) = s.subset(set_preimage_map(f, t))
    }
}

/// The closure of a source set induced by image followed by preimage.
define set_image_preimage_closure[T, U](f: T -> U, s: Set[T]) -> Set[T] {
    set_preimage(f, set_image(s, f))
}

/// Membership in the image-preimage closure is membership in the preimage of the image.
theorem set_image_preimage_closure_contains_eq[T, U](f: T -> U, s: Set[T], x: T) {
    set_image_preimage_closure(f, s).contains(x) =
        set_preimage(f, set_image(s, f)).contains(x)
}

/// The image-preimage closure is extensive.
theorem set_image_preimage_closure_extensive[T, U](f: T -> U, s: Set[T]) {
    s.subset(set_image_preimage_closure(f, s))
} by {
    subset_set_preimage_image(s, f)
    set_image_preimage_closure(f, s) = set_preimage(f, set_image(s, f))
}

/// The image-preimage closure is monotone.
theorem set_image_preimage_closure_monotone[T, U](f: T -> U, s: Set[T], t: Set[T]) {
    s.subset(t) implies
    set_image_preimage_closure(f, s).subset(set_image_preimage_closure(f, t))
} by {
    if s.subset(t) {
        forall(x: T) {
            if set_image_preimage_closure(f, s).contains(x) {
                set_image_preimage_closure_contains_eq(f, s, x)
                set_preimage_contains_eq(f, set_image(s, f), x)
                set_image(s, f).contains(f(x))
                set_image_contains_witness(s, f, f(x))
                let z: T satisfy {
                    s.contains(z) and f(x) = f(z)
                }
                t.contains(z)
                maps_into_set_image(t, f, z)
                set_image(t, f).contains(f(z))
                set_image(t, f).contains(f(x))
                set_preimage_contains_eq(f, set_image(t, f), x)
                set_preimage(f, set_image(t, f)).contains(x)
                set_image_preimage_closure_contains_eq(f, t, x)
                set_image_preimage_closure(f, t).contains(x)
            }
        }
    }
}

/// The image-preimage closure is idempotent.
theorem set_image_preimage_closure_idempotent[T, U](f: T -> U, s: Set[T]) {
    set_image_preimage_closure(f, set_image_preimage_closure(f, s)) =
        set_image_preimage_closure(f, s)
} by {
    let c = set_image_preimage_closure(f, s)
    let cc = set_image_preimage_closure(f, c)

    set_image_preimage_closure_extensive(f, c)
    c.subset(cc)

    forall(x: T) {
        if cc.contains(x) {
            set_image_preimage_closure_contains_eq(f, c, x)
            set_preimage_contains_eq(f, set_image(c, f), x)
            set_image(c, f).contains(f(x))
            set_image_contains_witness(c, f, f(x))
            let z: T satisfy {
                c.contains(z) and f(x) = f(z)
            }
            set_image_preimage_closure_contains_eq(f, s, z)
            set_preimage_contains_eq(f, set_image(s, f), z)
            set_image(s, f).contains(f(z))
            set_image(s, f).contains(f(x))
            set_preimage_contains_eq(f, set_image(s, f), x)
            set_preimage(f, set_image(s, f)).contains(x)
            set_image_preimage_closure_contains_eq(f, s, x)
            c.contains(x)
        }
    }
    cc.subset(c)
    double_inclusion(cc, c)
    cc = c
}

/// Injective maps have trivial image-preimage closure.
theorem set_image_preimage_closure_of_injective[T, U](f: T -> U, s: Set[T]) {
    is_injective_fn(f) implies set_image_preimage_closure(f, s) = s
} by {
    if is_injective_fn(f) {
        set_preimage_image_of_injective(s, f)
        set_image_preimage_closure(f, s) = set_preimage(f, set_image(s, f))
        set_image_preimage_closure(f, s) = s
    }
}

/// The image-preimage closure is a closure operator for inclusion.
theorem set_image_preimage_closure_is_set_closure_operator[T, U](f: T -> U) {
    is_set_closure_operator(set_image_preimage_closure(f))
} by {
    forall(s: Set[T], t: Set[T]) {
        if s.subset(t) {
            set_image_preimage_closure_monotone(f, s, t)
            set_image_preimage_closure(f, s).subset(set_image_preimage_closure(f, t))
        }
    }
    is_subset_monotone_map(set_image_preimage_closure(f))
    forall(s: Set[T]) {
        set_image_preimage_closure_extensive(f, s)
        s.subset(set_image_preimage_closure(f, s))
    }
    is_set_extensive_map(set_image_preimage_closure(f))
    forall(s: Set[T]) {
        set_image_preimage_closure_idempotent(f, s)
        set_image_preimage_closure(f, set_image_preimage_closure(f, s)) =
            set_image_preimage_closure(f, s)
    }
    is_set_idempotent_map(set_image_preimage_closure(f))
    is_set_closure_operator(set_image_preimage_closure(f))
}

/// The kernel of a target set induced by preimage followed by image.
define set_preimage_image_kernel[T, U](f: T -> U, s: Set[U]) -> Set[U] {
    set_image(set_preimage(f, s), f)
}

/// Membership in the preimage-image kernel is membership in the image of the preimage.
theorem set_preimage_image_kernel_contains_eq[T, U](f: T -> U, s: Set[U], y: U) {
    set_preimage_image_kernel(f, s).contains(y) =
        set_image(set_preimage(f, s), f).contains(y)
}

/// The preimage-image kernel is reductive.
theorem set_preimage_image_kernel_reductive[T, U](f: T -> U, s: Set[U]) {
    set_preimage_image_kernel(f, s).subset(s)
} by {
    set_image_preimage_subset(s, f)
    set_preimage_image_kernel(f, s) = set_image(set_preimage(f, s), f)
}

/// The preimage-image kernel is monotone.
theorem set_preimage_image_kernel_monotone[T, U](f: T -> U, s: Set[U], t: Set[U]) {
    s.subset(t) implies
    set_preimage_image_kernel(f, s).subset(set_preimage_image_kernel(f, t))
} by {
    if s.subset(t) {
        forall(y: U) {
            if set_preimage_image_kernel(f, s).contains(y) {
                set_preimage_image_kernel_contains_eq(f, s, y)
                set_image_contains_witness(set_preimage(f, s), f, y)
                let x: T satisfy {
                    set_preimage(f, s).contains(x) and y = f(x)
                }
                set_preimage_contains_eq(f, s, x)
                s.contains(f(x))
                t.contains(f(x))
                set_preimage_contains_eq(f, t, x)
                set_preimage(f, t).contains(x)
                maps_into_set_image(set_preimage(f, t), f, x)
                set_image(set_preimage(f, t), f).contains(f(x))
                set_image(set_preimage(f, t), f).contains(y)
                set_preimage_image_kernel_contains_eq(f, t, y)
                set_preimage_image_kernel(f, t).contains(y)
            }
        }
    }
}

/// The preimage-image kernel is idempotent.
theorem set_preimage_image_kernel_idempotent[T, U](f: T -> U, s: Set[U]) {
    set_preimage_image_kernel(f, set_preimage_image_kernel(f, s)) =
        set_preimage_image_kernel(f, s)
} by {
    let k = set_preimage_image_kernel(f, s)
    let kk = set_preimage_image_kernel(f, k)

    set_preimage_image_kernel_reductive(f, k)
    kk.subset(k)

    forall(y: U) {
        if k.contains(y) {
            set_preimage_image_kernel_contains_eq(f, s, y)
            set_image_contains_witness(set_preimage(f, s), f, y)
            let x: T satisfy {
                set_preimage(f, s).contains(x) and y = f(x)
            }
            set_preimage_contains_eq(f, k, x)
            k.contains(f(x))
            set_preimage(f, k).contains(x)
            maps_into_set_image(set_preimage(f, k), f, x)
            set_image(set_preimage(f, k), f).contains(f(x))
            set_image(set_preimage(f, k), f).contains(y)
            set_preimage_image_kernel_contains_eq(f, k, y)
            kk.contains(y)
        }
    }
    k.subset(kk)
    double_inclusion(kk, k)
    kk = k
}

/// Under surjectivity, taking image after preimage recovers the original set.
theorem set_image_preimage_of_surjective[T, U](s: Set[U], f: T -> U) {
    is_surjective_fn(f) implies set_image(set_preimage(f, s), f) = s
} by {
    let u = set_image(set_preimage(f, s), f)
    let v = s
    if is_surjective_fn(f) {
        set_image_preimage_subset(s, f)
        u.subset(v)
        forall(y: U) {
            if s.contains(y) {
                surjective_fn_has_preimage(f, y)
                let x: T satisfy {
                    f(x) = y
                }
                set_preimage_contains_eq(f, s, x)
                set_preimage(f, s).contains(x)
                maps_into_set_image(set_preimage(f, s), f, x)
                u.contains(f(x))
                u.contains(y)
            }
        }
        v.subset(u)
        double_inclusion(u, v)
        u = v
    }
}

/// Surjective maps have trivial preimage-image kernel.
theorem set_preimage_image_kernel_of_surjective[T, U](f: T -> U, s: Set[U]) {
    is_surjective_fn(f) implies set_preimage_image_kernel(f, s) = s
} by {
    if is_surjective_fn(f) {
        set_image_preimage_of_surjective(s, f)
        set_preimage_image_kernel(f, s) = set_image(set_preimage(f, s), f)
        set_preimage_image_kernel(f, s) = s
    }
}

/// The preimage-image kernel is a kernel operator for inclusion.
theorem set_preimage_image_kernel_is_set_kernel_operator[T, U](f: T -> U) {
    is_set_kernel_operator(set_preimage_image_kernel(f))
} by {
    forall(s: Set[U], t: Set[U]) {
        if s.subset(t) {
            set_preimage_image_kernel_monotone(f, s, t)
            set_preimage_image_kernel(f, s).subset(set_preimage_image_kernel(f, t))
        }
    }
    is_subset_monotone_map(set_preimage_image_kernel(f))
    forall(s: Set[U]) {
        set_preimage_image_kernel_reductive(f, s)
        set_preimage_image_kernel(f, s).subset(s)
    }
    is_set_reductive_map(set_preimage_image_kernel(f))
    forall(s: Set[U]) {
        set_preimage_image_kernel_idempotent(f, s)
        set_preimage_image_kernel(f, set_preimage_image_kernel(f, s)) =
            set_preimage_image_kernel(f, s)
    }
    is_set_idempotent_map(set_preimage_image_kernel(f))
    is_set_kernel_operator(set_preimage_image_kernel(f))
}

/// Under injectivity, image inclusion reflects to source inclusion.
theorem set_image_subset_imp_subset_of_injective[T, U](a: Set[T], b: Set[T], f: T -> U) {
    is_injective_fn(f) and set_image(a, f).subset(set_image(b, f)) implies a.subset(b)
} by {
    let u = set_image(a, f)
    let v = set_image(b, f)
    if is_injective_fn(f) and set_image(a, f).subset(set_image(b, f)) {
        if u.subset(v) {
            forall(x: T) {
                if a.contains(x) {
                    maps_into_set_image(a, f, x)
                    u.contains(f(x))
                    v.contains(f(x))
                    set_image_contains_witness(b, f, f(x))
                    let x2: T satisfy {
                        b.contains(x2) and f(x) = f(x2)
                    }
                    injective_fn_eq(f, x, x2)
                    x = x2
                    b.contains(x)
                }
            }
        }
    }
}

/// Under injectivity, equal images come from equal source sets.
theorem set_image_eq_imp_eq_of_injective[T, U](a: Set[T], b: Set[T], f: T -> U) {
    is_injective_fn(f) and set_image(a, f) = set_image(b, f) implies a = b
} by {
    if is_injective_fn(f) and set_image(a, f) = set_image(b, f) {
        set_image(a, f).subset(set_image(b, f))
        set_image_subset_imp_subset_of_injective(a, b, f)
        a.subset(b)
        set_image(b, f).subset(set_image(a, f))
        set_image_subset_imp_subset_of_injective(b, a, f)
        b.subset(a)
        double_inclusion(a, b)
        a = b
    }
}

/// Under surjectivity, preimage inclusion reflects to target inclusion.
theorem set_preimage_subset_imp_subset_of_surjective[T, U](a: Set[U], b: Set[U], f: T -> U) {
    is_surjective_fn(f) and set_preimage(f, a).subset(set_preimage(f, b)) implies a.subset(b)
} by {
    let u = set_preimage(f, a)
    let v = set_preimage(f, b)
    if is_surjective_fn(f) and set_preimage(f, a).subset(set_preimage(f, b)) {
        if u.subset(v) {
            forall(y: U) {
                if a.contains(y) {
                    surjective_fn_has_preimage(f, y)
                    let x: T satisfy {
                        f(x) = y
                    }
                    set_preimage_contains_eq(f, a, x)
                    u.contains(x)
                    v.contains(x)
                    set_preimage_contains_eq(f, b, x)
                    b.contains(f(x))
                    b.contains(y)
                }
            }
        }
    }
}

/// Under surjectivity, equal preimages come from equal target sets.
theorem set_preimage_eq_imp_eq_of_surjective[T, U](a: Set[U], b: Set[U], f: T -> U) {
    is_surjective_fn(f) and set_preimage(f, a) = set_preimage(f, b) implies a = b
} by {
    if is_surjective_fn(f) and set_preimage(f, a) = set_preimage(f, b) {
        set_preimage(f, a).subset(set_preimage(f, b))
        set_preimage_subset_imp_subset_of_surjective(a, b, f)
        a.subset(b)
        set_preimage(f, b).subset(set_preimage(f, a))
        set_preimage_subset_imp_subset_of_surjective(b, a, f)
        b.subset(a)
        double_inclusion(a, b)
        a = b
    }
}

// Difference theorems
// Basic properties of set difference
/// Membership in a difference is membership in the left set together with non-membership in the right set.
theorem difference_contains_eq[K](a: Set[K], b: Set[K], x: K) {
    a.difference(b).contains(x) = (a.contains(x) and not b.contains(x))
}

/// Membership in a difference gives membership in the left set.
theorem difference_contains_left[K](a: Set[K], b: Set[K], x: K) {
    a.difference(b).contains(x) implies a.contains(x)
} by {
    if a.difference(b).contains(x) {
        difference_contains_eq(a, b, x)
        a.contains(x)
    }
}

/// Membership in a difference gives non-membership in the right set.
theorem difference_contains_not_right[K](a: Set[K], b: Set[K], x: K) {
    a.difference(b).contains(x) implies not b.contains(x)
} by {
    if a.difference(b).contains(x) {
        difference_contains_eq(a, b, x)
        not b.contains(x)
    }
}

/// Membership in the left set and non-membership in the right set gives membership in the difference.
theorem difference_contains_intro[K](a: Set[K], b: Set[K], x: K) {
    a.contains(x) and not b.contains(x) implies a.difference(b).contains(x)
} by {
    if a.contains(x) and not b.contains(x) {
        difference_contains_eq(a, b, x)
        a.difference(b).contains(x)
    }
}

theorem difference_subset[K](a: Set[K], b: Set[K]) {
    a.difference(b).subset(a)
} by {
    forall(x: K) {
        if a.difference(b).contains(x) {
            difference_contains_eq(a, b, x)
            a.contains(x)
        }
    }
}

theorem difference_contains_imp_not_contains[K](a: Set[K], b: Set[K], x: K) {
    a.difference(b).contains(x) implies not b.contains(x)
}

theorem difference_contains_imp_contains[K](a: Set[K], b: Set[K], x: K) {
    a.difference(b).contains(x) implies a.contains(x)
}

theorem difference_contains_of_membership[K](a: Set[K], b: Set[K], x: K) {
    a.contains(x) and not b.contains(x) implies a.difference(b).contains(x)
}

theorem difference_of_self_is_empty[K](a: Set[K]) {
    a.difference(a) = Set[K].empty_set
} by {
    let d = a.difference(a)
    let e = Set[K].empty_set
    forall(x: K) {
        not d.contains(x)
    }
    d.subset(e)
}

// Insert/remove lemmas
theorem insert_contains[K](s: Set[K], item: K) {
    s.insert(item).contains(item)
}

theorem remove_does_not_contain[K](s: Set[K], item: K) {
    not s.remove(item).contains(item)
}

theorem insert_other_still_contains[K](s: Set[K], item: K, other: K) {
    s.contains(item) implies s.insert(other).contains(item)
}

theorem remove_other_still_contains[K](s: Set[K], item: K, other: K) {
    item != other and s.contains(item) implies s.remove(other).contains(item)
}

theorem insert_other_contains_imp_contains[K](s: Set[K], item: K, other: K) {
    item != other and s.insert(other).contains(item) implies s.contains(item)
}

theorem remove_other_contains_imp_contains[K](s: Set[K], item: K, other: K) {
    item != other and s.remove(other).contains(item) implies s.contains(item)
}

theorem insert_other_contains_eq[K](s: Set[K], item: K, other: K) {
    item != other implies s.contains(item) = s.insert(other).contains(item)
} by {
    if s.contains(item) {
    } else {
        s.contains(item) = s.insert(other).contains(item)
    }
}

theorem remove_other_contains_eq[K](s: Set[K], item: K, other: K) {
    item != other implies s.contains(item) = s.remove(other).contains(item)
} by {
    if s.contains(item) {
    } else {
        s.contains(item) = s.remove(other).contains(item)
    }
}

/// Membership after insertion is equality with the inserted element or prior membership.
theorem insert_contains_eq[K](s: Set[K], item: K, x: K) {
    s.insert(item).contains(x) = (x = item or s.contains(x))
} by {
    if x = item {
        insert_contains(s, item)
        s.insert(item).contains(x)
        x = item or s.contains(x)
        s.insert(item).contains(x) = (x = item or s.contains(x))
    } else {
        insert_other_contains_eq(s, x, item)
        s.contains(x) = s.insert(item).contains(x)
        s.insert(item).contains(x) = s.contains(x)
        s.insert(item).contains(x) = (x = item or s.contains(x))
    }
}

/// Membership after removal is prior membership together with inequality from the removed element.
theorem remove_contains_eq[K](s: Set[K], item: K, x: K) {
    s.remove(item).contains(x) = (s.contains(x) and x != item)
} by {
    if x = item {
        remove_does_not_contain(s, item)
        not s.remove(item).contains(x)
        not (s.contains(x) and x != item)
        s.remove(item).contains(x) = (s.contains(x) and x != item)
    } else {
        remove_other_contains_eq(s, x, item)
        s.contains(x) = s.remove(item).contains(x)
        s.remove(item).contains(x) = s.contains(x)
        s.remove(item).contains(x) = (s.contains(x) and x != item)
    }
}

theorem remove_then_insert[K](s: Set[K], item: K) {
    s.contains(item) implies s.remove(item).insert(item) = s
} by {
    if s.remove(item).insert(item) != s {
        s.remove(item).insert(item).contains != s.contains
        let t: K satisfy {
            s.remove(item).insert(item).contains(t) != s.contains(t)
        }
        if item != t {
            if s.contains(t) {
                false
            } else {
                false
            }
        } else {
            false
        }
    }
}

theorem insert_then_remove[K](s: Set[K], item: K) {
    not s.contains(item) implies s.insert(item).remove(item) = s
} by {
    if s.insert(item).remove(item) != s {
        s.insert(item).remove(item).contains != s.contains
        let t: K satisfy {
            s.insert(item).remove(item).contains(t) != s.contains(t)
        }
        if item != t {
            if s.contains(t) {
                false
            } else {
                false
            }
        } else {
            false
        }
    }
}

theorem union_with_difference_decomp[K](s: Set[K], t: Set[K]) {
    s.union(t) = t.union(s.difference(t))
} by {
    let u = s.union(t)
    let v = t.union(s.difference(t))

    forall(x: K) {
        if u.contains(x) {
            if s.contains(x) {
                if t.contains(x) {
                    elem_in_union(t, s.difference(t), x)
                } else {
                    s.difference(t).contains(x)
                    elem_in_union(t, s.difference(t), x)
                }
            } else {
                t.contains(x)
                elem_in_union(t, s.difference(t), x)
            }
            v.contains(x)
        }
    }
    u.subset(v)

    forall(x: K) {
        if v.contains(x) {
            if t.contains(x) {
                elem_in_union(s, t, x)
                u.contains(x)
            } else {
                s.difference(t).contains(x) implies s.contains(x)
                s.contains(x)
                elem_in_union(s, t, x)
                u.contains(x)
            }
        }
    }
    v.subset(u)
}

/// S ∪ (T \ S) = S ∪ T
theorem union_with_difference_decomp_rev[K](s: Set[K], t: Set[K]) {
    s.union(t) = s.union(t.difference(s))
} by {
    let u = s.union(t)
    let v = s.union(t.difference(s))

    forall(x: K) {
        if u.contains(x) {
            if s.contains(x) {
                elem_in_union(s, t.difference(s), x)
            } else {
                t.contains(x)
                t.difference(s).contains(x)
                elem_in_union(s, t.difference(s), x)
            }
            v.contains(x)
        }
    }
    u.subset(v)

    forall(x: K) {
        if v.contains(x) {
            if s.contains(x) {
                elem_in_union(s, t, x)
                u.contains(x)
            } else {
                t.difference(s.intersection(t)).contains(x) implies t.contains(x)
                t.contains(x)
                elem_in_union(s, t, x)
                u.contains(x)
            }
        }
    }
    v.subset(u)
}

/// S ∪ (T \ (S ∩ T)) = S ∪ T
theorem union_with_difference_decomp_inter[K](s: Set[K], t: Set[K]) {
    s.union(t) = s.union(t.difference(s.intersection(t)))
} by {
    let u = s.union(t)
    let v = s.union(t.difference(s.intersection(t)))

    forall(x: K) {
        if u.contains(x) {
            if s.contains(x) {
                elem_in_union(s, t.difference(s.intersection(t)), x)
                v.contains(x)
            } else {
                t.contains(x)
                not s.intersection(t).contains(x)
                t.difference(s.intersection(t)).contains(x)
                elem_in_union(s, t.difference(s.intersection(t)), x)
                v.contains(x)
            }
        }
    }
    u.subset(v)

    forall(x: K) {
        if v.contains(x) {
            if s.contains(x) {
                elem_in_union(s, t, x)
                u.contains(x)
            } else {
                t.difference(s.intersection(t)).contains(x) implies t.contains(x)
                t.contains(x)
                elem_in_union(s, t, x)
                u.contains(x)
            }
        }
    }
    v.subset(u)
}

// Union theorems
// A \subseteq A \cup B (and same with B)
/// Membership in a union is Boolean disjunction of membership.
theorem union_contains_eq[K](a: Set[K], b: Set[K], x: K) {
    a.union(b).contains(x) = (a.contains(x) or b.contains(x))
}

/// Membership in the left set gives membership in the union.
theorem union_contains_left[K](a: Set[K], b: Set[K], x: K) {
    a.contains(x) implies a.union(b).contains(x)
} by {
    if a.contains(x) {
        union_contains_eq(a, b, x)
        a.union(b).contains(x)
    }
}

/// Membership in the right set gives membership in the union.
theorem union_contains_right[K](a: Set[K], b: Set[K], x: K) {
    b.contains(x) implies a.union(b).contains(x)
} by {
    if b.contains(x) {
        union_contains_eq(a, b, x)
        a.union(b).contains(x)
    }
}

/// Membership in a union gives membership in one of its two sets.
theorem union_contains_cases[K](a: Set[K], b: Set[K], x: K) {
    a.union(b).contains(x) implies a.contains(x) or b.contains(x)
} by {
    if a.union(b).contains(x) {
        union_contains_eq(a, b, x)
        a.contains(x) or b.contains(x)
    }
}

/// Non-membership in a union is non-membership in both sets.
theorem not_union_contains_eq[K](a: Set[K], b: Set[K], x: K) {
    not a.union(b).contains(x) = (not a.contains(x) and not b.contains(x))
} by {
    if not a.union(b).contains(x) {
        if a.contains(x) {
            union_contains_left(a, b, x)
            false
        }
        if b.contains(x) {
            union_contains_right(a, b, x)
            false
        }
        not a.contains(x) and not b.contains(x)
    }
    if not a.contains(x) and not b.contains(x) {
        if a.union(b).contains(x) {
            union_contains_cases(a, b, x)
            if a.contains(x) {
                false
            } else {
                b.contains(x)
                false
            }
        }
        not a.union(b).contains(x)
    }
}

/// The successor initial segment is the previous one together with its endpoint.
theorem nat_lt_set_suc_eq_union(bound: Nat) {
    nat_lt_set(bound.suc) = nat_lt_set(bound).union(Set[Nat].singleton(bound))
} by {
    let u = nat_lt_set(bound.suc)
    let v = nat_lt_set(bound).union(Set[Nat].singleton(bound))
    forall(n: Nat) {
        if u.contains(n) {
            nat_lt_set_contains_eq(bound.suc, n)
            union_contains_eq(nat_lt_set(bound), Set[Nat].singleton(bound), n)
            if n = bound {
                singleton_contains_eq[Nat](bound, n)
                v.contains(n)
            } else {
                n < bound
                nat_lt_set_contains_eq(bound, n)
                nat_lt_set(bound).contains(n)
                v.contains(n)
            }
        }
        if v.contains(n) {
            union_contains_eq(nat_lt_set(bound), Set[Nat].singleton(bound), n)
            if nat_lt_set(bound).contains(n) {
                nat_lt_set_contains_eq(bound, n)
                nat_lt_set_contains_eq(bound.suc, n)
                u.contains(n)
            } else {
                singleton_contains_eq[Nat](bound, n)
                n = bound
                nat_lt_set_contains_eq(bound.suc, n)
                u.contains(n)
            }
        }
        u.contains(n) = v.contains(n)
    }
    set_ext(u, v)
}

/// Membership in an intersection is Boolean conjunction of membership.
theorem intersection_contains_eq[K](a: Set[K], b: Set[K], x: K) {
    a.intersection(b).contains(x) = (a.contains(x) and b.contains(x))
}

/// Membership in an intersection gives membership in the left set.
theorem intersection_contains_left[K](a: Set[K], b: Set[K], x: K) {
    a.intersection(b).contains(x) implies a.contains(x)
} by {
    if a.intersection(b).contains(x) {
        intersection_contains_eq(a, b, x)
        a.contains(x)
    }
}

/// Membership in an intersection gives membership in the right set.
theorem intersection_contains_right[K](a: Set[K], b: Set[K], x: K) {
    a.intersection(b).contains(x) implies b.contains(x)
} by {
    if a.intersection(b).contains(x) {
        intersection_contains_eq(a, b, x)
        b.contains(x)
    }
}

/// Membership in both sets gives membership in their intersection.
theorem intersection_contains_intro[K](a: Set[K], b: Set[K], x: K) {
    a.contains(x) and b.contains(x) implies a.intersection(b).contains(x)
} by {
    if a.contains(x) and b.contains(x) {
        intersection_contains_eq(a, b, x)
        a.intersection(b).contains(x)
    }
}

/// Non-membership in an intersection is failure of membership in at least one set.
theorem not_intersection_contains_eq[K](a: Set[K], b: Set[K], x: K) {
    not a.intersection(b).contains(x) = (not a.contains(x) or not b.contains(x))
} by {
    if not a.intersection(b).contains(x) {
        if a.contains(x) {
            if b.contains(x) {
                intersection_contains_intro(a, b, x)
                false
            }
            not b.contains(x)
            not a.contains(x) or not b.contains(x)
        } else {
            not a.contains(x)
            not a.contains(x) or not b.contains(x)
        }
    }
    if not a.contains(x) or not b.contains(x) {
        if a.intersection(b).contains(x) {
            intersection_contains_left(a, b, x)
            intersection_contains_right(a, b, x)
            if not a.contains(x) {
                false
            } else {
                not b.contains(x)
                false
            }
        }
        not a.intersection(b).contains(x)
    }
}

/// Image distributes over union.
theorem set_image_union[T, U](a: Set[T], b: Set[T], f: T -> U) {
    set_image(a.union(b), f) = set_image(a, f).union(set_image(b, f))
} by {
    let u = set_image(a.union(b), f)
    let v = set_image(a, f).union(set_image(b, f))
    forall(y: U) {
        if u.contains(y) {
            let x: T satisfy {
                a.union(b).contains(x) and y = f(x)
            }
            union_contains_eq(a, b, x)
            if a.contains(x) {
                maps_into_set_image(a, f, x)
                set_image(a, f).contains(f(x))
                set_image(a, f).contains(y)
            } else {
                b.contains(x)
                maps_into_set_image(b, f, x)
                set_image(b, f).contains(f(x))
                set_image(b, f).contains(y)
            }
            union_contains_eq(set_image(a, f), set_image(b, f), y)
            v.contains(y)
        }
        if v.contains(y) {
            union_contains_eq(set_image(a, f), set_image(b, f), y)
            if set_image(a, f).contains(y) {
                let x: T satisfy {
                    a.contains(x) and y = f(x)
                }
                union_contains_eq(a, b, x)
                a.union(b).contains(x)
                image_contains(f, a.union(b), y)
                u.contains(y)
            } else {
                let x: T satisfy {
                    b.contains(x) and y = f(x)
                }
                union_contains_eq(a, b, x)
                a.union(b).contains(x)
                image_contains(f, a.union(b), y)
                u.contains(y)
            }
        }
        u.contains(y) = v.contains(y)
    }
    set_ext(u, v)
}

/// The image of an intersection is contained in the intersection of the images.
theorem set_image_intersection_subset[T, U](a: Set[T], b: Set[T], f: T -> U) {
    set_image(a.intersection(b), f).subset(set_image(a, f).intersection(set_image(b, f)))
} by {
    forall(y: U) {
        if set_image(a.intersection(b), f).contains(y) {
            let x: T satisfy {
                a.intersection(b).contains(x) and y = f(x)
            }
            intersection_contains_eq(a, b, x)
            a.contains(x)
            b.contains(x)
            maps_into_set_image(a, f, x)
            set_image(a, f).contains(f(x))
            set_image(a, f).contains(y)
            maps_into_set_image(b, f, x)
            set_image(b, f).contains(f(x))
            set_image(b, f).contains(y)
            intersection_contains_eq(set_image(a, f), set_image(b, f), y)
            set_image(a, f).intersection(set_image(b, f)).contains(y)
        }
    }
}

/// Under injectivity, the intersection of the images comes from the image of the intersection.
theorem set_image_intersection_reverse_of_injective[T, U](a: Set[T], b: Set[T], f: T -> U) {
    (forall(x1: T, x2: T) { f(x1) = f(x2) implies x1 = x2 })
    implies
    set_image(a, f).intersection(set_image(b, f)).subset(set_image(a.intersection(b), f))
} by {
    if forall(x1: T, x2: T) { f(x1) = f(x2) implies x1 = x2 } {
        forall(y: U) {
            if set_image(a, f).intersection(set_image(b, f)).contains(y) {
                intersection_contains_eq(set_image(a, f), set_image(b, f), y)
                set_image(a, f).contains(y)
                set_image_contains_witness(a, f, y)
                let x1: T satisfy {
                    a.contains(x1) and y = f(x1)
                }
                set_image(b, f).contains(y)
                set_image_contains_witness(b, f, y)
                let x2: T satisfy {
                    b.contains(x2) and y = f(x2)
                }
                y = f(x1)
                y = f(x2)
                f(x1) = f(x2)
                x1 = x2
                b.contains(x1)
                intersection_contains_eq(a, b, x1)
                a.intersection(b).contains(x1)
                image_contains(f, a.intersection(b), y)
                set_image(a.intersection(b), f).contains(y)
            }
        }
    }
}

/// Under injectivity, the image of an intersection is exactly the intersection of the images.
theorem set_image_intersection_of_injective[T, U](a: Set[T], b: Set[T], f: T -> U) {
    (forall(x1: T, x2: T) { f(x1) = f(x2) implies x1 = x2 })
    implies
    set_image(a.intersection(b), f) = set_image(a, f).intersection(set_image(b, f))
} by {
    if forall(x1: T, x2: T) { f(x1) = f(x2) implies x1 = x2 } {
        set_image_intersection_subset(a, b, f)
        set_image_intersection_reverse_of_injective(a, b, f)
        set_image(a.intersection(b), f).subset(set_image(a, f).intersection(set_image(b, f)))
        set_image(a, f).intersection(set_image(b, f)).subset(set_image(a.intersection(b), f))
        double_inclusion(set_image(a.intersection(b), f), set_image(a, f).intersection(set_image(b, f)))
        set_image(a.intersection(b), f) = set_image(a, f).intersection(set_image(b, f))
    }
}

/// Under injectivity, the image of a difference is contained in the difference of the images.
theorem set_image_difference_subset_of_injective[T, U](a: Set[T], b: Set[T], f: T -> U) {
    (forall(x1: T, x2: T) { f(x1) = f(x2) implies x1 = x2 })
    implies
    set_image(a.difference(b), f).subset(set_image(a, f).difference(set_image(b, f)))
} by {
    if forall(x1: T, x2: T) { f(x1) = f(x2) implies x1 = x2 } {
        forall(y: U) {
            if set_image(a.difference(b), f).contains(y) {
                set_image_contains_witness(a.difference(b), f, y)
                let x: T satisfy {
                    a.difference(b).contains(x) and y = f(x)
                }
                difference_contains_eq(a, b, x)
                a.contains(x)
                not b.contains(x)
                maps_into_set_image(a, f, x)
                set_image(a, f).contains(f(x))
                set_image(a, f).contains(y)
                if set_image(b, f).contains(y) {
                    set_image_contains_witness(b, f, y)
                    let xb: T satisfy {
                        b.contains(xb) and y = f(xb)
                    }
                    y = f(x)
                    y = f(xb)
                    f(x) = f(xb)
                    x = xb
                    b.contains(x)
                    false
                }
                difference_contains_eq(set_image(a, f), set_image(b, f), y)
                set_image(a, f).difference(set_image(b, f)).contains(y)
            }
        }
    }
}

/// Under injectivity, the difference of the images comes from the image of the difference.
theorem set_image_difference_reverse_of_injective[T, U](a: Set[T], b: Set[T], f: T -> U) {
    (forall(x1: T, x2: T) { f(x1) = f(x2) implies x1 = x2 })
    implies
    set_image(a, f).difference(set_image(b, f)).subset(set_image(a.difference(b), f))
} by {
    if forall(x1: T, x2: T) { f(x1) = f(x2) implies x1 = x2 } {
        forall(y: U) {
            if set_image(a, f).difference(set_image(b, f)).contains(y) {
                difference_contains_eq(set_image(a, f), set_image(b, f), y)
                set_image(a, f).contains(y)
                not set_image(b, f).contains(y)
                set_image_contains_witness(a, f, y)
                let x: T satisfy {
                    a.contains(x) and y = f(x)
                }
                if b.contains(x) {
                    maps_into_set_image(b, f, x)
                    set_image(b, f).contains(f(x))
                    set_image(b, f).contains(y)
                    false
                }
                difference_contains_of_membership(a, b, x)
                a.difference(b).contains(x)
                maps_into_set_image(a.difference(b), f, x)
                set_image(a.difference(b), f).contains(f(x))
                set_image(a.difference(b), f).contains(y)
            }
        }
    }
}

/// Under injectivity, the image of a difference is the difference of the images.
theorem set_image_difference_of_injective[T, U](a: Set[T], b: Set[T], f: T -> U) {
    (forall(x1: T, x2: T) { f(x1) = f(x2) implies x1 = x2 })
    implies
    set_image(a.difference(b), f) = set_image(a, f).difference(set_image(b, f))
} by {
    if forall(x1: T, x2: T) { f(x1) = f(x2) implies x1 = x2 } {
        set_image_difference_subset_of_injective(a, b, f)
        set_image_difference_reverse_of_injective(a, b, f)
        set_image(a.difference(b), f).subset(set_image(a, f).difference(set_image(b, f)))
        set_image(a, f).difference(set_image(b, f)).subset(set_image(a.difference(b), f))
        double_inclusion(set_image(a.difference(b), f), set_image(a, f).difference(set_image(b, f)))
        set_image(a.difference(b), f) = set_image(a, f).difference(set_image(b, f))
    }
}

/// A surjective map hits every element of the codomain from the universal set.
theorem set_image_universal_contains_of_surjective[T, U](f: T -> U, y: U) {
    is_surjective_fn(f) implies set_image(Set[T].universal_set, f).contains(y)
} by {
    if is_surjective_fn(f) {
        surjective_fn_has_preimage(f, y)
        let x: T satisfy {
            f(x) = y
        }
        universal_set_contains_eq[T](x)
        Set[T].universal_set.contains(x)
        maps_into_set_image(Set[T].universal_set, f, x)
        set_image(Set[T].universal_set, f).contains(f(x))
        set_image(Set[T].universal_set, f).contains(y)
    }
}

/// A surjective map sends the universal set to the universal set.
theorem set_image_universal_of_surjective[T, U](f: T -> U) {
    is_surjective_fn(f) implies set_image(Set[T].universal_set, f) = Set[U].universal_set
} by {
    if is_surjective_fn(f) {
        set_preimage_universal(f)
        set_image_preimage_of_surjective(Set[U].universal_set, f)
        set_image(set_preimage(f, Set[U].universal_set), f) = Set[U].universal_set
        set_image(Set[T].universal_set, f) = Set[U].universal_set
    }
}

/// A bijection sends the image of a complement into the complement of the image.
theorem set_image_compl_subset_of_bijection[T, U](s: Set[T], f: T -> U) {
    is_bijection_fn(f) implies set_image(s.c, f).subset(set_image(s, f).c)
} by {
    if is_bijection_fn(f) {
        bijection_fn_is_injective(f)
        is_injective_fn(f)
        forall(y: U) {
            if set_image(s.c, f).contains(y) {
                set_image_contains_witness(s.c, f, y)
                let x1: T satisfy {
                    s.c.contains(x1) and y = f(x1)
                }
                compl_contains_eq(s, x1)
                not s.contains(x1)
                if set_image(s, f).contains(y) {
                    set_image_contains_witness(s, f, y)
                    let x2: T satisfy {
                        s.contains(x2) and y = f(x2)
                    }
                    y = f(x1)
                    y = f(x2)
                    f(x1) = f(x2)
                    injective_fn_eq(f, x1, x2)
                    x1 = x2
                    s.contains(x1)
                    false
                }
                compl_contains_eq(set_image(s, f), y)
                set_image(s, f).c.contains(y)
            }
        }
    }
}

/// A bijection sends the complement of the image back into the image of the complement.
theorem set_image_compl_reverse_of_bijection[T, U](s: Set[T], f: T -> U) {
    is_bijection_fn(f) implies set_image(s, f).c.subset(set_image(s.c, f))
} by {
    if is_bijection_fn(f) {
        bijection_fn_is_surjective(f)
        is_surjective_fn(f)
        forall(y: U) {
            if set_image(s, f).c.contains(y) {
                compl_contains_eq(set_image(s, f), y)
                not set_image(s, f).contains(y)
                surjective_fn_has_preimage(f, y)
                let x: T satisfy {
                    f(x) = y
                }
                if s.contains(x) {
                    maps_into_set_image(s, f, x)
                    set_image(s, f).contains(f(x))
                    set_image(s, f).contains(y)
                    false
                }
                compl_contains_eq(s, x)
                s.c.contains(x)
                maps_into_set_image(s.c, f, x)
                set_image(s.c, f).contains(f(x))
                set_image(s.c, f).contains(y)
            }
        }
    }
}

/// A bijection sends complements to complements.
theorem set_image_compl_of_bijection[T, U](s: Set[T], f: T -> U) {
    is_bijection_fn(f) implies set_image(s.c, f) = set_image(s, f).c
} by {
    if is_bijection_fn(f) {
        set_image_compl_subset_of_bijection(s, f)
        set_image_compl_reverse_of_bijection(s, f)
        set_image(s.c, f).subset(set_image(s, f).c)
        set_image(s, f).c.subset(set_image(s.c, f))
        double_inclusion(set_image(s.c, f), set_image(s, f).c)
        set_image(s.c, f) = set_image(s, f).c
    }
}

/// Preimage distributes over union.
theorem set_preimage_union[T, U](f: T -> U, a: Set[U], b: Set[U]) {
    set_preimage(f, a.union(b)) = set_preimage(f, a).union(set_preimage(f, b))
} by {
    let u = set_preimage(f, a.union(b))
    let v = set_preimage(f, a).union(set_preimage(f, b))
    forall(x: T) {
        set_preimage_contains_eq(f, a.union(b), x)
        union_contains_eq(a, b, f(x))
        set_preimage_contains_eq(f, a, x)
        set_preimage_contains_eq(f, b, x)
        union_contains_eq(set_preimage(f, a), set_preimage(f, b), x)
        u.contains(x) = v.contains(x)
    }
    set_ext(u, v)
}

/// Preimage distributes over intersection.
theorem set_preimage_intersection[T, U](f: T -> U, a: Set[U], b: Set[U]) {
    set_preimage(f, a.intersection(b)) = set_preimage(f, a).intersection(set_preimage(f, b))
} by {
    let u = set_preimage(f, a.intersection(b))
    let v = set_preimage(f, a).intersection(set_preimage(f, b))
    forall(x: T) {
        set_preimage_contains_eq(f, a.intersection(b), x)
        intersection_contains_eq(a, b, f(x))
        set_preimage_contains_eq(f, a, x)
        set_preimage_contains_eq(f, b, x)
        intersection_contains_eq(set_preimage(f, a), set_preimage(f, b), x)
        u.contains(x) = v.contains(x)
    }
    set_ext(u, v)
}

theorem sets_subset_union[K](a: Set[K], b: Set[K]) {
    a.subset(a.union(b)) and b.subset(a.union(b))
} by {
    forall(x: K) {
        if a.contains(x) {
            union_contains_eq(a, b, x)
            a.union(b).contains(x)
        }
    }
    a.subset(a.union(b))
    forall(x: K) {
        if b.contains(x) {
            union_contains_eq(a, b, x)
            a.union(b).contains(x)
        }
    }
    b.subset(a.union(b))
}

// A \subseteq C and B \subseteq C implies A \cup B \subseteq C
theorem sets_subset_contain_union[K](a: Set[K], b: Set[K], c: Set[K]) {
    a.subset(c) and b.subset(c) implies a.union(b).subset(c)
} by {
    if a.subset(c) and b.subset(c) {
        forall(x: K) {
            if a.union(b).contains(x) {
                elem_in_union(a, b, x)
                if a.contains(x) {
                    c.contains(x)
                } else {
                    b.contains(x)
                    c.contains(x)
                }
            }
        }
    }
}

theorem union_comm[K](a: Set[K], b: Set[K]) {
    a.union(b) = b.union(a)
} by {
    let u = a.union(b)
    let v = b.union(a)

}

theorem union_assoc[K](a: Set[K], b: Set[K], c: Set[K]) {
    a.union(b.union(c)) = a.union(b).union(c)
} by {
    let u = a.union(b.union(c))
    let v = a.union(b).union(c)

    // Note that A, B, C are subsets of U hence V is a subset of U
    a.subset(u)
    b.union(c).subset(u)
    b.subset(b.union(c))
    b.subset(u)
    c.subset(b.union(c))
    c.subset(u)
    a.union(b).subset(u)
    v.subset(u)

    // Reverse
    a.union(b).subset(v)
    a.subset(a.union(b))
    a.subset(v)
    b.subset(a.union(b))
    b.subset(v)
    c.subset(v)
    b.union(c).subset(v)
    u.subset(v)
}

theorem union_idemp[K](s: Set[K]) {
    s.union(s) = s
}

theorem union_subset_is_set[K](a: Set[K], b: Set[K]) {
    a.subset(b) implies a.union(b) = b
}

theorem union_with_empty_is_self[K](s: Set[K]) {
    s.union(Set[K].empty_set) = s
}

theorem union_with_universal_is_universal[K](s: Set[K]) {
    s.union(Set[K].universal_set) = Set[K].universal_set
}

/// A set united with its complement is universal.
theorem union_compl_is_universal[K](s: Set[K]) {
    s.union(s.c) = Set[K].universal_set
} by {
    let u = s.union(s.c)
    let v = Set[K].universal_set
    forall(x: K) {
        if u.contains(x) {
            universal_set_contains_eq[K](x)
            v.contains(x)
        }
        if v.contains(x) {
            universal_set_contains_eq[K](x)
            if s.contains(x) {
                union_contains_eq(s, s.c, x)
                u.contains(x)
            } else {
                compl_contains_eq(s, x)
                s.c.contains(x)
                union_contains_eq(s, s.c, x)
                u.contains(x)
            }
        }
        u.contains(x) = v.contains(x)
    }
    set_ext(u, v)
}

// Intersection theorems
theorem sets_subset_intersection[K](a: Set[K], b: Set[K]) {
    a.intersection(b).subset(a) and a.intersection(b).subset(b)
} by {
    forall(x: K) {
        if a.intersection(b).contains(x) {
            intersection_contains_eq(a, b, x)
            a.contains(x)
        }
    }
    a.intersection(b).subset(a)
    forall(x: K) {
        if a.intersection(b).contains(x) {
            intersection_contains_eq(a, b, x)
            b.contains(x)
        }
    }
    a.intersection(b).subset(b)
}

theorem set_supset_contains_intersection[K](a: Set[K], b: Set[K], c: Set[K]) {
    a.superset(c) and b.superset(c) implies a.intersection(b).superset(c)
} by {
    if a.superset(c) and b.superset(c) {
        forall(x: K) {
            if c.contains(x) {
                a.contains(x)
                b.contains(x)
                elem_in_intersection(a, b, x)
                a.intersection(b).contains(x)
            }
        }
    }
}

theorem intersection_comm[K](a: Set[K], b: Set[K]) {
    a.intersection(b) = b.intersection(a)
} by {
    let u = a.intersection(b)
    let v = b.intersection(a)

    forall(x: K) {
        if u.contains(x) {
            elem_in_intersection(a, b, x)
            a.contains(x)
            b.contains(x)
            elem_in_intersection(b, a, x)
            v.contains(x)
        }
    }
    u.subset(v)

    forall(x: K) {
        if v.contains(x) {
            elem_in_intersection(b, a, x)
            a.contains(x)
            b.contains(x)
            elem_in_intersection(a, b, x)
            u.contains(x)
        }
    }
    v.subset(u)
}

theorem intersection_assoc[K](a: Set[K], b: Set[K], c: Set[K]) {
    a.intersection(b.intersection(c)) = a.intersection(b).intersection(c)
} by {
    let u = a.intersection(b.intersection(c))
    let v = a.intersection(b).intersection(c)

    // See union proof for comments
    a.superset(u)
    b.intersection(c).superset(u)
    b.superset(b.intersection(c))
    b.superset(u)
    c.superset(b.intersection(c))
    c.superset(u)
    a.intersection(b).superset(u)
    v.superset(u)

    a.intersection(b).superset(v)
    a.superset(a.intersection(b))
    a.superset(v)
    b.superset(a.intersection(b))
    b.superset(v)
    c.superset(v)
    b.intersection(c).superset(v)
    u.superset(v)
}

theorem intersection_idemp[K](s: Set[K]) {
    s.intersection(s) = s
}

theorem intersection_with_superset_is_self[K](s: Set[K], t: Set[K]) {
    s.subset(t) implies s.intersection(t) = s
} by {
    if s.subset(t) {
        s.intersection(t).subset(s)
        s.superset(s)
        t.superset(s)
        s.intersection(t).superset(s)
        s.subset(s.intersection(t))
    }
}

theorem intersection_with_universal_is_self[K](s: Set[K]) {
    s.intersection(Set[K].universal_set) = s
}

theorem intersection_with_empty_is_empty[K](s: Set[K]) {
    s.intersection(Set[K].empty_set) = Set[K].empty_set
}

// Union + intersection theorems
theorem union_intersection_distrib[K](a: Set[K], b: Set[K], c: Set[K]) {
    a.union(b.intersection(c)) = a.union(b).intersection(a.union(c))
} by {
    let u = a.union(b.intersection(c))
    let v = a.union(b).intersection(a.union(c))

    // Forward direction ("easy")
    a.subset(a.union(b))
    a.union(b).superset(a)
    a.subset(a.union(c))
    a.union(c).superset(a)
    v.superset(a)
    a.subset(v)

    let bc = b.intersection(c)
    bc.subset(b)
    b.subset(a.union(b))
    bc.subset(a.union(b))
    a.union(b).superset(bc)
    bc.subset(c)
    c.subset(a.union(c))
    bc.subset(a.union(c))
    a.union(c).superset(bc)
    v.superset(bc)
    bc.subset(v)

    // Other direction
    forall(x: K) {
        if v.contains(x) {
            a.union(b).contains(x)
            a.union(c).contains(x)

            if a.contains(x) {
                elem_in_union(a, b.intersection(c), x)
                u.contains(x)
            } else {
                b.contains(x)
                c.contains(x)
                elem_in_intersection(b, c, x)
                b.intersection(c).contains(x)
                elem_in_union(a, b.intersection(c), x)
                u.contains(x)
            }
        }
    }
    v.subset(u)
}

// Union and intersection over a family of sets
define or_family[K, I] (f: I -> Set[K], x: K) -> Bool {
    exists(i: I) {
        f(i).contains(x)
    }
}

define and_family[K, I] (f: I -> Set[K], x: K) -> Bool {
    forall(i: I) {
        f(i).contains(x)
    }
}

define union_family[K, I](f: I -> Set[K]) -> Set[K] {
    Set[K].new(or_family(f))
}

define intersection_family[K, I](f: I -> Set[K]) -> Set[K] {
    Set[K].new(and_family(f))
}

/// The union of an indexed family of sets.
define indexed_union[K, I](family: I -> Set[K]) -> Set[K] {
    union_family(family)
}

/// The intersection of an indexed family of sets.
define indexed_intersection[K, I](family: I -> Set[K]) -> Set[K] {
    intersection_family(family)
}

/// Membership in an indexed union is membership in some member of the family.
theorem indexed_union_contains_eq[K, I](family: I -> Set[K], x: K) {
    indexed_union(family).contains(x) = exists(i: I) {
        family(i).contains(x)
    }
} by {
    indexed_union(family) = union_family(family)
    union_family(family).contains(x) = or_family(family, x)
}

/// Membership in a member of a family gives membership in its indexed union.
theorem indexed_union_contains_of_contains[K, I](family: I -> Set[K], i: I, x: K) {
    family(i).contains(x) implies indexed_union(family).contains(x)
} by {
    if family(i).contains(x) {
        exists(j: I) {
            j = i and family(j).contains(x)
        }
        indexed_union_contains_eq(family, x)
        indexed_union(family).contains(x)
    }
}

/// Membership in an indexed union has a witnessing index.
theorem indexed_union_contains_witness[K, I](family: I -> Set[K], x: K) {
    indexed_union(family).contains(x) implies exists(i: I) {
        family(i).contains(x)
    }
} by {
    if indexed_union(family).contains(x) {
        indexed_union_contains_eq(family, x)
        exists(i: I) {
            family(i).contains(x)
        }
    }
}

/// Membership in an indexed intersection is membership in every member of the family.
theorem indexed_intersection_contains_eq[K, I](family: I -> Set[K], x: K) {
    indexed_intersection(family).contains(x) = forall(i: I) {
        family(i).contains(x)
    }
} by {
    indexed_intersection(family) = intersection_family(family)
    intersection_family(family).contains(x) = and_family(family, x)
}

/// Membership in an indexed intersection gives membership in each member of the family.
theorem indexed_intersection_contains_at[K, I](family: I -> Set[K], i: I, x: K) {
    indexed_intersection(family).contains(x) implies family(i).contains(x)
} by {
    if indexed_intersection(family).contains(x) {
        indexed_intersection_contains_eq(family, x)
        family(i).contains(x)
    }
}

/// Membership in every member of a family gives membership in its indexed intersection.
theorem indexed_intersection_contains_of_forall[K, I](family: I -> Set[K], x: K) {
    (forall(i: I) { family(i).contains(x) }) implies indexed_intersection(family).contains(x)
} by {
    if forall(i: I) { family(i).contains(x) } {
        indexed_intersection_contains_eq(family, x)
        indexed_intersection(family).contains(x)
    }
}

/// The indexed union is contained in any common superset of the family.
theorem indexed_union_subset_of_family_subset[K, I](family: I -> Set[K], s: Set[K]) {
    (forall(i: I) { family(i).subset(s) }) implies indexed_union(family).subset(s)
} by {
    if forall(i: I) { family(i).subset(s) } {
        forall(x: K) {
            if indexed_union(family).contains(x) {
                indexed_union_contains_witness(family, x)
                let i: I satisfy {
                    family(i).contains(x)
                }
                family(i).subset(s)
                family(i).subset(s) = forall(y: K) {
                    family(i).contains(y) implies s.contains(y)
                }
                family(i).contains(x) implies s.contains(x)
                s.contains(x)
            }
        }
    }
}

/// The indexed intersection contains any common subset of the family.
theorem indexed_intersection_superset_of_subset_family[K, I](family: I -> Set[K], s: Set[K]) {
    (forall(i: I) { s.subset(family(i)) }) implies indexed_intersection(family).superset(s)
} by {
    if forall(i: I) { s.subset(family(i)) } {
        forall(x: K) {
            if s.contains(x) {
                forall(i: I) {
                    s.subset(family(i))
                    s.subset(family(i)) = forall(y: K) {
                        s.contains(y) implies family(i).contains(y)
                    }
                    s.contains(x) implies family(i).contains(x)
                    family(i).contains(x)
                }
                indexed_intersection_contains_of_forall(family, x)
                indexed_intersection(family).contains(x)
            }
        }
    }
}

/// Pointwise inclusion of indexed families preserves indexed unions.
theorem indexed_union_monotone[K, I](a: I -> Set[K], b: I -> Set[K]) {
    (forall(i: I) { a(i).subset(b(i)) }) implies indexed_union(a).subset(indexed_union(b))
} by {
    if forall(i: I) { a(i).subset(b(i)) } {
        forall(x: K) {
            if indexed_union(a).contains(x) {
                indexed_union_contains_witness(a, x)
                let i: I satisfy {
                    a(i).contains(x)
                }
                a(i).subset(b(i))
                a(i).subset(b(i)) = forall(y: K) {
                    a(i).contains(y) implies b(i).contains(y)
                }
                a(i).contains(x) implies b(i).contains(x)
                b(i).contains(x)
                indexed_union_contains_of_contains(b, i, x)
                indexed_union(b).contains(x)
            }
        }
    }
}

/// Pointwise inclusion of indexed families preserves indexed intersections.
theorem indexed_intersection_monotone[K, I](a: I -> Set[K], b: I -> Set[K]) {
    (forall(i: I) { a(i).subset(b(i)) }) implies
    indexed_intersection(a).subset(indexed_intersection(b))
} by {
    if forall(i: I) { a(i).subset(b(i)) } {
        forall(x: K) {
            if indexed_intersection(a).contains(x) {
                forall(i: I) {
                    indexed_intersection_contains_at(a, i, x)
                    a(i).contains(x)
                    a(i).subset(b(i))
                    a(i).subset(b(i)) = forall(y: K) {
                        a(i).contains(y) implies b(i).contains(y)
                    }
                    a(i).contains(x) implies b(i).contains(x)
                    b(i).contains(x)
                }
                indexed_intersection_contains_of_forall(b, x)
                indexed_intersection(b).contains(x)
            }
        }
    }
}

/// An indexed union of empty sets is empty.
theorem indexed_union_empty_of_forall_empty[K, I](family: I -> Set[K]) {
    (forall(i: I) { family(i).is_empty }) implies
    indexed_union(family) = Set[K].empty_set
} by {
    if forall(i: I) { family(i).is_empty } {
        forall(x: K) {
            if indexed_union(family).contains(x) {
                indexed_union_contains_witness(family, x)
                let i: I satisfy {
                    family(i).contains(x)
                }
                family(i).is_empty
                false
            }
        }
        set_eq_empty_of_is_empty(indexed_union(family))
    }
}

/// If an indexed union is empty, every member of the family is empty.
theorem indexed_union_empty_imp_forall_empty[K, I](family: I -> Set[K]) {
    indexed_union(family) = Set[K].empty_set implies forall(i: I) {
        family(i).is_empty
    }
} by {
    if indexed_union(family) = Set[K].empty_set {
        forall(i: I) {
            forall(x: K) {
                if family(i).contains(x) {
                    indexed_union_contains_of_contains(family, i, x)
                    indexed_union(family).contains(x)
                    Set[K].empty_set.contains(x)
                    empty_set_contains_eq[K](x)
                    false
                }
            }
            family(i).is_empty
        }
    }
}

/// An indexed intersection of universal sets is universal.
theorem indexed_intersection_universal_of_forall_universal[K, I](family: I -> Set[K]) {
    (forall(i: I) { family(i) = Set[K].universal_set }) implies
    indexed_intersection(family) = Set[K].universal_set
} by {
    if forall(i: I) { family(i) = Set[K].universal_set } {
        forall(x: K) {
            forall(i: I) {
                family(i) = Set[K].universal_set
                universal_set_contains_eq[K](x)
                family(i).contains(x)
            }
            indexed_intersection_contains_of_forall(family, x)
            indexed_intersection(family).contains(x)
        }
        set_eq_universal_of_forall_contains(indexed_intersection(family))
    }
}

/// If an indexed intersection is universal, every member of the family is universal.
theorem indexed_intersection_universal_imp_forall_universal[K, I](family: I -> Set[K]) {
    indexed_intersection(family) = Set[K].universal_set implies forall(i: I) {
        family(i) = Set[K].universal_set
    }
} by {
    if indexed_intersection(family) = Set[K].universal_set {
        forall(i: I) {
            forall(x: K) {
                universal_set_contains_eq[K](x)
                indexed_intersection(family).contains(x)
                indexed_intersection_contains_at(family, i, x)
                family(i).contains(x)
            }
            set_eq_universal_of_forall_contains(family(i))
        }
    }
}

/// The complement of an indexed union is the indexed intersection of the complements.
theorem indexed_union_complement_eq_intersection_complement[K, I](family: I -> Set[K]) {
    indexed_union(function(i: I) { family(i).c }) = indexed_intersection(family).c
} by {
    let u = indexed_union(function(i: I) { family(i).c })
    let v = indexed_intersection(family).c
    forall(x: K) {
        if u.contains(x) {
            indexed_union_contains_witness(function(i: I) { family(i).c }, x)
            let i: I satisfy {
                family(i).c.contains(x)
            }
            compl_contains_eq(family(i), x)
            not family(i).contains(x)
            not indexed_intersection(family).contains(x)
            compl_contains_eq(indexed_intersection(family), x)
            v.contains(x)
        }
        if v.contains(x) {
            compl_contains_eq(indexed_intersection(family), x)
            not indexed_intersection(family).contains(x)
            indexed_intersection_contains_eq(family, x)
            let i: I satisfy {
                not family(i).contains(x)
            }
            compl_contains_eq(family(i), x)
            family(i).c.contains(x)
            indexed_union_contains_of_contains(function(j: I) { family(j).c }, i, x)
            u.contains(x)
        }
        u.contains(x) = v.contains(x)
    }
    set_ext(u, v)
}

/// The complement of an indexed intersection is the indexed union of the complements.
theorem indexed_intersection_complement_eq_union_complement[K, I](family: I -> Set[K]) {
    indexed_intersection(function(i: I) { family(i).c }) = indexed_union(family).c
} by {
    let u = indexed_intersection(function(i: I) { family(i).c })
    let v = indexed_union(family).c
    forall(x: K) {
        if u.contains(x) {
            forall(i: I) {
                indexed_intersection_contains_at(function(j: I) { family(j).c }, i, x)
                family(i).c.contains(x)
                compl_contains_eq(family(i), x)
                not family(i).contains(x)
            }
            if indexed_union(family).contains(x) {
                indexed_union_contains_witness(family, x)
                let i: I satisfy {
                    family(i).contains(x)
                }
                not family(i).contains(x)
                false
            }
            not indexed_union(family).contains(x)
            compl_contains_eq(indexed_union(family), x)
            v.contains(x)
        }
        if v.contains(x) {
            compl_contains_eq(indexed_union(family), x)
            not indexed_union(family).contains(x)
            forall(i: I) {
                not family(i).contains(x)
                compl_contains_eq(family(i), x)
                family(i).c.contains(x)
            }
            indexed_intersection_contains_of_forall(function(i: I) { family(i).c }, x)
            u.contains(x)
        }
        u.contains(x) = v.contains(x)
    }
    set_ext(u, v)
}

/// Preimage commutes with indexed union.
theorem set_preimage_indexed_union[T, U, I](f: T -> U, family: I -> Set[U]) {
    set_preimage(f, indexed_union(family)) =
        indexed_union(function(i: I) { set_preimage(f, family(i)) })
} by {
    let u = set_preimage(f, indexed_union(family))
    let v = indexed_union(function(i: I) { set_preimage(f, family(i)) })
    forall(x: T) {
        if u.contains(x) {
            set_preimage_contains_eq(f, indexed_union(family), x)
            indexed_union(family).contains(f(x))
            indexed_union_contains_witness(family, f(x))
            let i: I satisfy { family(i).contains(f(x)) }
            set_preimage_contains_eq(f, family(i), x)
            set_preimage(f, family(i)).contains(x)
            indexed_union_contains_of_contains(
                function(j: I) { set_preimage(f, family(j)) }, i, x)
            v.contains(x)
        }
        if v.contains(x) {
            indexed_union_contains_witness(
                function(j: I) { set_preimage(f, family(j)) }, x)
            let i: I satisfy { set_preimage(f, family(i)).contains(x) }
            set_preimage_contains_eq(f, family(i), x)
            family(i).contains(f(x))
            indexed_union_contains_of_contains(family, i, f(x))
            indexed_union(family).contains(f(x))
            set_preimage_contains_eq(f, indexed_union(family), x)
            u.contains(x)
        }
        u.contains(x) = v.contains(x)
    }
    set_ext(u, v)
}

/// Preimage commutes with indexed intersection.
theorem set_preimage_indexed_intersection[T, U, I](f: T -> U, family: I -> Set[U]) {
    set_preimage(f, indexed_intersection(family)) =
        indexed_intersection(function(i: I) { set_preimage(f, family(i)) })
} by {
    let u = set_preimage(f, indexed_intersection(family))
    let v = indexed_intersection(function(i: I) { set_preimage(f, family(i)) })
    forall(x: T) {
        if u.contains(x) {
            set_preimage_contains_eq(f, indexed_intersection(family), x)
            indexed_intersection(family).contains(f(x))
            forall(i: I) {
                indexed_intersection_contains_at(family, i, f(x))
                family(i).contains(f(x))
                set_preimage_contains_eq(f, family(i), x)
                set_preimage(f, family(i)).contains(x)
            }
            indexed_intersection_contains_of_forall(
                function(j: I) { set_preimage(f, family(j)) }, x)
            v.contains(x)
        }
        if v.contains(x) {
            forall(i: I) {
                indexed_intersection_contains_at(
                    function(j: I) { set_preimage(f, family(j)) }, i, x)
                set_preimage(f, family(i)).contains(x)
                set_preimage_contains_eq(f, family(i), x)
                family(i).contains(f(x))
            }
            indexed_intersection_contains_of_forall(family, f(x))
            indexed_intersection(family).contains(f(x))
            set_preimage_contains_eq(f, indexed_intersection(family), x)
            u.contains(x)
        }
        u.contains(x) = v.contains(x)
    }
    set_ext(u, v)
}

/// Membership in the union of a list-indexed family.
define list_indexed_union_contains[K, I](items: List[I], family: I -> Set[K], x: K) -> Bool {
    exists(i: I) {
        items.contains(i) and family(i).contains(x)
    }
}

/// The union of the members whose indices occur in a list.
define list_indexed_union[K, I](items: List[I], family: I -> Set[K]) -> Set[K] {
    Set[K].new(list_indexed_union_contains(items, family))
}

/// Membership in the intersection of a list-indexed family.
define list_indexed_intersection_contains[K, I](items: List[I], family: I -> Set[K], x: K) -> Bool {
    forall(i: I) {
        items.contains(i) implies family(i).contains(x)
    }
}

/// The intersection of the members whose indices occur in a list.
define list_indexed_intersection[K, I](items: List[I], family: I -> Set[K]) -> Set[K] {
    Set[K].new(list_indexed_intersection_contains(items, family))
}

/// Membership in a list-indexed union is witnessed by an index in the list.
theorem list_indexed_union_contains_eq[K, I](items: List[I], family: I -> Set[K], x: K) {
    list_indexed_union(items, family).contains(x) = exists(i: I) {
        items.contains(i) and family(i).contains(x)
    }
}

/// A listed index contributes its member set to the list-indexed union.
theorem list_indexed_union_contains_of_contains[K, I](items: List[I], family: I -> Set[K],
    i: I, x: K) {
    items.contains(i) and family(i).contains(x) implies list_indexed_union(items, family).contains(x)
} by {
    if items.contains(i) and family(i).contains(x) {
        exists(j: I) {
            j = i and items.contains(j) and family(j).contains(x)
        }
        list_indexed_union_contains_eq(items, family, x)
        list_indexed_union(items, family).contains(x)
    }
}

/// Membership in a list-indexed union has a listed witnessing index.
theorem list_indexed_union_contains_witness[K, I](items: List[I], family: I -> Set[K], x: K) {
    list_indexed_union(items, family).contains(x) implies exists(i: I) {
        items.contains(i) and family(i).contains(x)
    }
} by {
    if list_indexed_union(items, family).contains(x) {
        list_indexed_union_contains_eq(items, family, x)
        exists(i: I) {
            items.contains(i) and family(i).contains(x)
        }
    }
}

/// Membership in a list-indexed intersection is membership in every listed member.
theorem list_indexed_intersection_contains_eq[K, I](items: List[I], family: I -> Set[K], x: K) {
    list_indexed_intersection(items, family).contains(x) = forall(i: I) {
        items.contains(i) implies family(i).contains(x)
    }
}

/// A member of a list-indexed intersection belongs to each listed member.
theorem list_indexed_intersection_contains_at[K, I](items: List[I], family: I -> Set[K],
    i: I, x: K) {
    list_indexed_intersection(items, family).contains(x) and items.contains(i)
    implies family(i).contains(x)
} by {
    if list_indexed_intersection(items, family).contains(x) and items.contains(i) {
        list_indexed_intersection_contains_eq(items, family, x)
        items.contains(i) implies family(i).contains(x)
        family(i).contains(x)
    }
}

/// Membership in every listed member gives membership in the list-indexed intersection.
theorem list_indexed_intersection_contains_of_forall[K, I](items: List[I], family: I -> Set[K],
    x: K) {
    (forall(i: I) { items.contains(i) implies family(i).contains(x) }) implies
    list_indexed_intersection(items, family).contains(x)
} by {
    if forall(i: I) { items.contains(i) implies family(i).contains(x) } {
        list_indexed_intersection_contains_eq(items, family, x)
        list_indexed_intersection(items, family).contains(x)
    }
}

/// A list-indexed union is contained in any common superset of its listed members.
theorem list_indexed_union_subset_of_family_subset[K, I](items: List[I], family: I -> Set[K],
    s: Set[K]) {
    (forall(i: I) { items.contains(i) implies family(i).subset(s) }) implies
    list_indexed_union(items, family).subset(s)
} by {
    if forall(i: I) { items.contains(i) implies family(i).subset(s) } {
        forall(x: K) {
            if list_indexed_union(items, family).contains(x) {
                list_indexed_union_contains_witness(items, family, x)
                let i: I satisfy {
                    items.contains(i) and family(i).contains(x)
                }
                family(i).subset(s)
                family(i).subset(s) = forall(y: K) {
                    family(i).contains(y) implies s.contains(y)
                }
                family(i).contains(x) implies s.contains(x)
                s.contains(x)
            }
        }
    }
}

/// A common subset of the listed members is contained in their list-indexed intersection.
theorem subset_list_indexed_intersection_of_subset_family[K, I](items: List[I],
    family: I -> Set[K], s: Set[K]) {
    (forall(i: I) { items.contains(i) implies s.subset(family(i)) }) implies
    s.subset(list_indexed_intersection(items, family))
} by {
    if forall(i: I) { items.contains(i) implies s.subset(family(i)) } {
        forall(x: K) {
            if s.contains(x) {
                forall(i: I) {
                    if items.contains(i) {
                        s.subset(family(i))
                        s.subset(family(i)) = forall(y: K) {
                            s.contains(y) implies family(i).contains(y)
                        }
                        s.contains(x) implies family(i).contains(x)
                        family(i).contains(x)
                    }
                }
                list_indexed_intersection_contains_of_forall(items, family, x)
                list_indexed_intersection(items, family).contains(x)
            }
        }
    }
}

/// Pointwise inclusion of listed members preserves list-indexed unions.
theorem list_indexed_union_monotone[K, I](items: List[I], a: I -> Set[K], b: I -> Set[K]) {
    (forall(i: I) { items.contains(i) implies a(i).subset(b(i)) }) implies
    list_indexed_union(items, a).subset(list_indexed_union(items, b))
} by {
    if forall(i: I) { items.contains(i) implies a(i).subset(b(i)) } {
        forall(x: K) {
            if list_indexed_union(items, a).contains(x) {
                list_indexed_union_contains_witness(items, a, x)
                let i: I satisfy {
                    items.contains(i) and a(i).contains(x)
                }
                a(i).subset(b(i))
                a(i).subset(b(i)) = forall(y: K) {
                    a(i).contains(y) implies b(i).contains(y)
                }
                a(i).contains(x) implies b(i).contains(x)
                b(i).contains(x)
                list_indexed_union_contains_of_contains(items, b, i, x)
                list_indexed_union(items, b).contains(x)
            }
        }
    }
}

/// Pointwise inclusion of listed members preserves list-indexed intersections.
theorem list_indexed_intersection_monotone[K, I](items: List[I], a: I -> Set[K],
    b: I -> Set[K]) {
    (forall(i: I) { items.contains(i) implies a(i).subset(b(i)) }) implies
    list_indexed_intersection(items, a).subset(list_indexed_intersection(items, b))
} by {
    if forall(i: I) { items.contains(i) implies a(i).subset(b(i)) } {
        forall(x: K) {
            if list_indexed_intersection(items, a).contains(x) {
                forall(i: I) {
                    if items.contains(i) {
                        list_indexed_intersection_contains_at(items, a, i, x)
                        a(i).contains(x)
                        a(i).subset(b(i))
                        a(i).subset(b(i)) = forall(y: K) {
                            a(i).contains(y) implies b(i).contains(y)
                        }
                        a(i).contains(x) implies b(i).contains(x)
                        b(i).contains(x)
                    }
                }
                list_indexed_intersection_contains_of_forall(items, b, x)
                list_indexed_intersection(items, b).contains(x)
            }
        }
    }
}

/// The list-indexed union is contained in the full indexed union.
theorem list_indexed_union_subset_indexed_union[K, I](items: List[I], family: I -> Set[K]) {
    list_indexed_union(items, family).subset(indexed_union(family))
} by {
    forall(x: K) {
        if list_indexed_union(items, family).contains(x) {
            list_indexed_union_contains_witness(items, family, x)
            let i: I satisfy {
                items.contains(i) and family(i).contains(x)
            }
            indexed_union_contains_of_contains(family, i, x)
            indexed_union(family).contains(x)
        }
    }
}

/// The full indexed intersection is contained in any list-indexed intersection.
theorem indexed_intersection_subset_list_indexed_intersection[K, I](items: List[I],
    family: I -> Set[K]) {
    indexed_intersection(family).subset(list_indexed_intersection(items, family))
} by {
    forall(x: K) {
        if indexed_intersection(family).contains(x) {
            forall(i: I) {
                if items.contains(i) {
                    indexed_intersection_contains_at(family, i, x)
                    family(i).contains(x)
                }
            }
            list_indexed_intersection_contains_of_forall(items, family, x)
            list_indexed_intersection(items, family).contains(x)
        }
    }
}

/// The union of the members whose natural indices are less than `n`.
define range_indexed_union[K](n: Nat, family: Nat -> Set[K]) -> Set[K] {
    list_indexed_union(n.range, family)
}

/// The intersection of the members whose natural indices are less than `n`.
define range_indexed_intersection[K](n: Nat, family: Nat -> Set[K]) -> Set[K] {
    list_indexed_intersection(n.range, family)
}

/// Membership in a range-indexed union is witnessed by an index below the bound.
theorem range_indexed_union_contains_eq[K](n: Nat, family: Nat -> Set[K], x: K) {
    range_indexed_union(n, family).contains(x) = exists(i: Nat) {
        i < n and family(i).contains(x)
    }
} by {
    let u = range_indexed_union(n, family)
    let v = list_indexed_union(n.range, family)
    u = v
    if u.contains(x) {
        list_indexed_union_contains_witness(n.range, family, x)
        let i: Nat satisfy {
            n.range.contains(i) and family(i).contains(x)
        }
        if not i < n {
            i >= n
            range_does_not_contain_geq(n, i)
            not n.range.contains(i)
            false
        }
        i < n
        exists(j: Nat) {
            j = i and j < n and family(j).contains(x)
        }
    }
    if exists(i: Nat) {
        i < n and family(i).contains(x)
    } {
        let i: Nat satisfy {
            i < n and family(i).contains(x)
        }
        range_contains_all_leq(n)
        n.range.contains(i)
        list_indexed_union_contains_of_contains(n.range, family, i, x)
        v.contains(x)
        u.contains(x)
    }
    u.contains(x) = exists(i: Nat) {
        i < n and family(i).contains(x)
    }
}

/// A member below the bound contributes to the range-indexed union.
theorem range_indexed_union_contains_of_lt[K](n: Nat, family: Nat -> Set[K], i: Nat, x: K) {
    i < n and family(i).contains(x) implies range_indexed_union(n, family).contains(x)
} by {
    if i < n and family(i).contains(x) {
        exists(j: Nat) {
            j = i and j < n and family(j).contains(x)
        }
        range_indexed_union_contains_eq(n, family, x)
        range_indexed_union(n, family).contains(x)
    }
}

/// Membership in a range-indexed union has an index below the bound.
theorem range_indexed_union_contains_witness[K](n: Nat, family: Nat -> Set[K], x: K) {
    range_indexed_union(n, family).contains(x) implies exists(i: Nat) {
        i < n and family(i).contains(x)
    }
} by {
    if range_indexed_union(n, family).contains(x) {
        range_indexed_union_contains_eq(n, family, x)
        exists(i: Nat) {
            i < n and family(i).contains(x)
        }
    }
}

/// Membership in a range-indexed intersection is membership in every term below the bound.
theorem range_indexed_intersection_contains_eq[K](n: Nat, family: Nat -> Set[K], x: K) {
    range_indexed_intersection(n, family).contains(x) = forall(i: Nat) {
        i < n implies family(i).contains(x)
    }
} by {
    let u = range_indexed_intersection(n, family)
    let v = list_indexed_intersection(n.range, family)
    u = v
    if u.contains(x) {
        forall(i: Nat) {
            if i < n {
                range_contains_all_leq(n)
                n.range.contains(i)
                list_indexed_intersection_contains_at(n.range, family, i, x)
                family(i).contains(x)
            }
        }
    }
    if forall(i: Nat) { i < n implies family(i).contains(x) } {
        forall(i: Nat) {
            if n.range.contains(i) {
                if not i < n {
                    i >= n
                    range_does_not_contain_geq(n, i)
                    not n.range.contains(i)
                    false
                }
                i < n
                family(i).contains(x)
            }
        }
        list_indexed_intersection_contains_of_forall(n.range, family, x)
        v.contains(x)
        u.contains(x)
    }
    u.contains(x) = forall(i: Nat) {
        i < n implies family(i).contains(x)
    }
}

/// A member of a range-indexed intersection belongs to each term below the bound.
theorem range_indexed_intersection_contains_at[K](n: Nat, family: Nat -> Set[K],
    i: Nat, x: K) {
    range_indexed_intersection(n, family).contains(x) and i < n implies family(i).contains(x)
} by {
    if range_indexed_intersection(n, family).contains(x) and i < n {
        range_indexed_intersection_contains_eq(n, family, x)
        i < n implies family(i).contains(x)
        family(i).contains(x)
    }
}

/// Membership in every term below the bound gives membership in the range-indexed intersection.
theorem range_indexed_intersection_contains_of_forall[K](n: Nat, family: Nat -> Set[K],
    x: K) {
    (forall(i: Nat) { i < n implies family(i).contains(x) }) implies
    range_indexed_intersection(n, family).contains(x)
} by {
    if forall(i: Nat) { i < n implies family(i).contains(x) } {
        range_indexed_intersection_contains_eq(n, family, x)
        range_indexed_intersection(n, family).contains(x)
    }
}

/// The union over an empty range is empty.
theorem range_indexed_union_zero[K](family: Nat -> Set[K]) {
    range_indexed_union(Nat.0, family) = Set[K].empty_set
} by {
    let u = range_indexed_union(Nat.0, family)
    let e = Set[K].empty_set
    forall(x: K) {
        if u.contains(x) {
            range_indexed_union_contains_witness(Nat.0, family, x)
            let i: Nat satisfy {
                i < Nat.0 and family(i).contains(x)
            }
            false
        }
        empty_set_contains_eq[K](x)
        u.contains(x) = e.contains(x)
    }
    set_ext(u, e)
}

/// The intersection over an empty range is universal.
theorem range_indexed_intersection_zero[K](family: Nat -> Set[K]) {
    range_indexed_intersection(Nat.0, family) = Set[K].universal_set
} by {
    let u = range_indexed_intersection(Nat.0, family)
    let v = Set[K].universal_set
    forall(x: K) {
        if u.contains(x) {
            universal_set_contains_eq[K](x)
            v.contains(x)
        }
        if v.contains(x) {
            forall(i: Nat) {
                if i < Nat.0 {
                    false
                }
            }
            range_indexed_intersection_contains_of_forall(Nat.0, family, x)
            u.contains(x)
        }
        u.contains(x) = v.contains(x)
    }
    set_ext(u, v)
}

/// The range-indexed union is contained in any common superset below the bound.
theorem range_indexed_union_subset_of_family_subset[K](n: Nat, family: Nat -> Set[K],
    s: Set[K]) {
    (forall(i: Nat) { i < n implies family(i).subset(s) }) implies
    range_indexed_union(n, family).subset(s)
} by {
    if forall(i: Nat) { i < n implies family(i).subset(s) } {
        forall(x: K) {
            if range_indexed_union(n, family).contains(x) {
                range_indexed_union_contains_witness(n, family, x)
                let i: Nat satisfy {
                    i < n and family(i).contains(x)
                }
                family(i).subset(s)
                family(i).subset(s) = forall(y: K) {
                    family(i).contains(y) implies s.contains(y)
                }
                family(i).contains(x) implies s.contains(x)
                s.contains(x)
            }
        }
    }
}

/// A common subset of all members below the bound is contained in their intersection.
theorem subset_range_indexed_intersection_of_subset_family[K](n: Nat,
    family: Nat -> Set[K], s: Set[K]) {
    (forall(i: Nat) { i < n implies s.subset(family(i)) }) implies
    s.subset(range_indexed_intersection(n, family))
} by {
    if forall(i: Nat) { i < n implies s.subset(family(i)) } {
        forall(x: K) {
            if s.contains(x) {
                forall(i: Nat) {
                    if i < n {
                        s.subset(family(i))
                        s.subset(family(i)) = forall(y: K) {
                            s.contains(y) implies family(i).contains(y)
                        }
                        s.contains(x) implies family(i).contains(x)
                        family(i).contains(x)
                    }
                }
                range_indexed_intersection_contains_of_forall(n, family, x)
                range_indexed_intersection(n, family).contains(x)
            }
        }
    }
}

/// Pointwise inclusion below the bound preserves range-indexed unions.
theorem range_indexed_union_monotone[K](n: Nat, a: Nat -> Set[K], b: Nat -> Set[K]) {
    (forall(i: Nat) { i < n implies a(i).subset(b(i)) }) implies
    range_indexed_union(n, a).subset(range_indexed_union(n, b))
} by {
    if forall(i: Nat) { i < n implies a(i).subset(b(i)) } {
        forall(x: K) {
            if range_indexed_union(n, a).contains(x) {
                range_indexed_union_contains_witness(n, a, x)
                let i: Nat satisfy {
                    i < n and a(i).contains(x)
                }
                a(i).subset(b(i))
                a(i).subset(b(i)) = forall(y: K) {
                    a(i).contains(y) implies b(i).contains(y)
                }
                a(i).contains(x) implies b(i).contains(x)
                b(i).contains(x)
                range_indexed_union_contains_of_lt(n, b, i, x)
                range_indexed_union(n, b).contains(x)
            }
        }
    }
}

/// Pointwise inclusion below the bound preserves range-indexed intersections.
theorem range_indexed_intersection_monotone[K](n: Nat, a: Nat -> Set[K],
    b: Nat -> Set[K]) {
    (forall(i: Nat) { i < n implies a(i).subset(b(i)) }) implies
    range_indexed_intersection(n, a).subset(range_indexed_intersection(n, b))
} by {
    if forall(i: Nat) { i < n implies a(i).subset(b(i)) } {
        forall(x: K) {
            if range_indexed_intersection(n, a).contains(x) {
                forall(i: Nat) {
                    if i < n {
                        range_indexed_intersection_contains_at(n, a, i, x)
                        a(i).contains(x)
                        a(i).subset(b(i))
                        a(i).subset(b(i)) = forall(y: K) {
                            a(i).contains(y) implies b(i).contains(y)
                        }
                        a(i).contains(x) implies b(i).contains(x)
                        b(i).contains(x)
                    }
                }
                range_indexed_intersection_contains_of_forall(n, b, x)
                range_indexed_intersection(n, b).contains(x)
            }
        }
    }
}

/// The complement of a bounded union is the bounded intersection of the complements.
theorem range_indexed_union_complement_eq_intersection_complement[K](n: Nat,
    family: Nat -> Set[K]) {
    range_indexed_union(n, function(i: Nat) { family(i).c }) =
    range_indexed_intersection(n, family).c
} by {
    let u = range_indexed_union(n, function(i: Nat) { family(i).c })
    let v = range_indexed_intersection(n, family).c
    forall(x: K) {
        if u.contains(x) {
            range_indexed_union_contains_witness(n, function(i: Nat) { family(i).c }, x)
            let i: Nat satisfy {
                i < n and family(i).c.contains(x)
            }
            compl_contains_eq(family(i), x)
            not family(i).contains(x)
            if range_indexed_intersection(n, family).contains(x) {
                range_indexed_intersection_contains_at(n, family, i, x)
                family(i).contains(x)
                false
            }
            not range_indexed_intersection(n, family).contains(x)
            v.contains(x)
        }
        if v.contains(x) {
            not range_indexed_intersection(n, family).contains(x)
            let r = range_indexed_intersection(n, family)
            if not exists(i: Nat) {
                i < n and not family(i).contains(x)
            } {
                forall(i: Nat) {
                    if i < n {
                        family(i).contains(x)
                    }
                }
                range_indexed_intersection_contains_of_forall(n, family, x)
                r.contains(x)
                false
            }
            let i: Nat satisfy {
                i < n and not family(i).contains(x)
            }
            compl_contains_eq(family(i), x)
            family(i).c.contains(x)
            range_indexed_union_contains_of_lt(n, function(j: Nat) { family(j).c }, i, x)
            u.contains(x)
        }
        u.contains(x) = v.contains(x)
    }
    set_ext(u, v)
}

/// The complement of a bounded intersection is the bounded union of the complements.
theorem range_indexed_intersection_complement_eq_union_complement[K](n: Nat,
    family: Nat -> Set[K]) {
    range_indexed_intersection(n, function(i: Nat) { family(i).c }) =
    range_indexed_union(n, family).c
} by {
    let u = range_indexed_intersection(n, function(i: Nat) { family(i).c })
    let v = range_indexed_union(n, family).c
    forall(x: K) {
        if u.contains(x) {
            forall(i: Nat) {
                if i < n {
                    range_indexed_intersection_contains_at(n, function(j: Nat) { family(j).c },
                        i, x)
                    family(i).c.contains(x)
                    compl_contains_eq(family(i), x)
                    not family(i).contains(x)
                }
            }
            if range_indexed_union(n, family).contains(x) {
                range_indexed_union_contains_witness(n, family, x)
                let i: Nat satisfy {
                    i < n and family(i).contains(x)
                }
                not family(i).contains(x)
                false
            }
            not range_indexed_union(n, family).contains(x)
            v.contains(x)
        }
        if v.contains(x) {
            not range_indexed_union(n, family).contains(x)
            forall(i: Nat) {
                if i < n {
                    if family(i).contains(x) {
                        range_indexed_union_contains_of_lt(n, family, i, x)
                        range_indexed_union(n, family).contains(x)
                        false
                    }
                    not family(i).contains(x)
                    compl_contains_eq(family(i), x)
                    family(i).c.contains(x)
                }
            }
            range_indexed_intersection_contains_of_forall(n, function(j: Nat) { family(j).c }, x)
            u.contains(x)
        }
        u.contains(x) = v.contains(x)
    }
    set_ext(u, v)
}

/// Increasing the range bound can only enlarge the range-indexed union.
theorem range_indexed_union_subset_of_bound_le[K](m: Nat, n: Nat, family: Nat -> Set[K]) {
    m <= n implies range_indexed_union(m, family).subset(range_indexed_union(n, family))
} by {
    if m <= n {
        forall(x: K) {
            if range_indexed_union(m, family).contains(x) {
                range_indexed_union_contains_witness(m, family, x)
                let i: Nat satisfy {
                    i < m and family(i).contains(x)
                }
                i < n
                range_indexed_union_contains_of_lt(n, family, i, x)
                range_indexed_union(n, family).contains(x)
            }
        }
    }
}

/// Increasing the range bound can only shrink the range-indexed intersection.
theorem range_indexed_intersection_subset_of_bound_le[K](m: Nat, n: Nat,
    family: Nat -> Set[K]) {
    m <= n implies range_indexed_intersection(n, family).subset(range_indexed_intersection(m, family))
} by {
    if m <= n {
        forall(x: K) {
            if range_indexed_intersection(n, family).contains(x) {
                forall(i: Nat) {
                    if i < m {
                        i < n
                        range_indexed_intersection_contains_at(n, family, i, x)
                        family(i).contains(x)
                    }
                }
                range_indexed_intersection_contains_of_forall(m, family, x)
                range_indexed_intersection(m, family).contains(x)
            }
        }
    }
}

// Sequences of sets
/// True if a sequence of sets is decreasing, meaning each set contains the next.
define is_decreasing[T](seq: Nat -> Set[T]) -> Bool {
    forall(i: Nat) {
        seq(i.suc).subset(seq(i))
    }
}

/// The intersection of all sets in a sequence.
define seq_intersection[T](seq: Nat -> Set[T]) -> Set[T] {
    intersection_family(seq)
}

/// True if a sequence of sets is increasing, meaning each set is contained in the next.
define is_increasing[T](seq: Nat -> Set[T]) -> Bool {
    forall(i: Nat) {
        seq(i).subset(seq(i.suc))
    }
}

/// The union of all sets in a sequence.
define seq_union[T](seq: Nat -> Set[T]) -> Set[T] {
    union_family(seq)
}

/// The sequence obtained by taking the image of each set under a fixed function.
define set_image_seq[T, U](seq: Nat -> Set[T], f: T -> U, n: Nat) -> Set[U] {
    set_image(seq(n), f)
}

/// The sequence where each set is replaced by its complement.
define seq_complement[T](seq: Nat -> Set[T]) -> (Nat -> Set[T]) {
    function(i: Nat) {
        seq(i).c
    }
}

/// The complement of an increasing sequence is decreasing.
theorem seq_complement_is_decreasing_of_increasing[T](seq: Nat -> Set[T]) {
    is_increasing(seq) implies is_decreasing(seq_complement(seq))
} by {
    if is_increasing(seq) {
        forall(i: Nat) {
            seq(i).subset(seq(i.suc))
            forall(x: T) {
                if seq_complement(seq)(i.suc).contains(x) {
                    compl_contains_eq(seq(i.suc), x)
                    not seq(i.suc).contains(x)
                    if seq(i).contains(x) {
                        seq(i).subset(seq(i.suc)) = forall(y: T) {
                            seq(i).contains(y) implies seq(i.suc).contains(y)
                        }
                        seq(i).contains(x) implies seq(i.suc).contains(x)
                        seq(i.suc).contains(x)
                        false
                    }
                    compl_contains_eq(seq(i), x)
                    seq_complement(seq)(i).contains(x)
                }
            }
            seq_complement(seq)(i.suc).subset(seq_complement(seq)(i))
        }
        is_decreasing(seq_complement(seq))
    }
}

/// The complement of a stepwise increasing sequence is decreasing.
theorem seq_complement_decreasing_of_step_subset[T](seq: Nat -> Set[T]) {
    (forall(i: Nat) { seq(i).subset(seq(i.suc)) }) implies
    is_decreasing(seq_complement(seq))
} by {
    if forall(i: Nat) { seq(i).subset(seq(i.suc)) } {
        is_increasing(seq)
        seq_complement_is_decreasing_of_increasing(seq)
        is_decreasing(seq_complement(seq))
    }
}

/// The complement of a decreasing sequence is increasing.
theorem seq_complement_is_increasing_of_decreasing[T](seq: Nat -> Set[T]) {
    is_decreasing(seq) implies is_increasing(seq_complement(seq))
} by {
    if is_decreasing(seq) {
        forall(i: Nat) {
            seq(i.suc).subset(seq(i))
            forall(x: T) {
                if seq_complement(seq)(i).contains(x) {
                    compl_contains_eq(seq(i), x)
                    not seq(i).contains(x)
                    if seq(i.suc).contains(x) {
                        seq(i.suc).subset(seq(i)) = forall(y: T) {
                            seq(i.suc).contains(y) implies seq(i).contains(y)
                        }
                        seq(i.suc).contains(x) implies seq(i).contains(x)
                        seq(i).contains(x)
                        false
                    }
                    compl_contains_eq(seq(i.suc), x)
                    seq_complement(seq)(i.suc).contains(x)
                }
            }
            seq_complement(seq)(i).subset(seq_complement(seq)(i.suc))
        }
        is_increasing(seq_complement(seq))
    }
}

/// The complement of a stepwise decreasing sequence is increasing.
theorem seq_complement_increasing_of_step_superset[T](seq: Nat -> Set[T]) {
    (forall(i: Nat) { seq(i.suc).subset(seq(i)) }) implies
    is_increasing(seq_complement(seq))
} by {
    if forall(i: Nat) { seq(i.suc).subset(seq(i)) } {
        is_decreasing(seq)
        seq_complement_is_increasing_of_decreasing(seq)
        is_increasing(seq_complement(seq))
    }
}

/// Applying sequence complement twice recovers the original sequence.
theorem seq_complement_complement[T](seq: Nat -> Set[T]) {
    seq_complement(seq_complement(seq)) = seq
} by {
    forall(i: Nat) {
        seq_complement(seq_complement(seq))(i) = seq_complement(seq)(i).c
        seq_complement(seq)(i) = seq(i).c
        seq_complement(seq)(i).c = seq(i).c.c
        compl_of_compl_is_self(seq(i))
        seq_complement(seq_complement(seq))(i) = seq(i)
    }
}

/// Membership in a sequence union is membership in some term.
theorem seq_union_contains_eq[T](seq: Nat -> Set[T], x: T) {
    seq_union(seq).contains(x) = exists(i: Nat) {
        seq(i).contains(x)
    }
} by {
    seq_union(seq) = union_family(seq)
    indexed_union(seq) = union_family(seq)
    indexed_union_contains_eq(seq, x)
}

/// Membership in a term of a sequence gives membership in the sequence union.
theorem seq_union_contains_of_contains[T](seq: Nat -> Set[T], i: Nat, x: T) {
    seq(i).contains(x) implies seq_union(seq).contains(x)
} by {
    if seq(i).contains(x) {
        indexed_union_contains_of_contains(seq, i, x)
        indexed_union(seq).contains(x)
        indexed_union(seq) = union_family(seq)
        seq_union(seq) = union_family(seq)
        seq_union(seq).contains(x)
    }
}

/// Membership in a sequence union has an index witness.
theorem seq_union_contains_witness[T](seq: Nat -> Set[T], x: T) {
    seq_union(seq).contains(x) implies exists(i: Nat) {
        seq(i).contains(x)
    }
} by {
    if seq_union(seq).contains(x) {
        seq_union_contains_eq(seq, x)
        exists(i: Nat) {
            seq(i).contains(x)
        }
    }
}

/// Membership in a sequence intersection is membership in every term.
theorem seq_intersection_contains_eq[T](seq: Nat -> Set[T], x: T) {
    seq_intersection(seq).contains(x) = forall(i: Nat) {
        seq(i).contains(x)
    }
} by {
    seq_intersection(seq) = intersection_family(seq)
    indexed_intersection(seq) = intersection_family(seq)
    indexed_intersection_contains_eq(seq, x)
}

/// Membership in a sequence intersection gives membership in each term.
theorem seq_intersection_contains_at[T](seq: Nat -> Set[T], i: Nat, x: T) {
    seq_intersection(seq).contains(x) implies seq(i).contains(x)
} by {
    if seq_intersection(seq).contains(x) {
        seq_intersection_contains_eq(seq, x)
        seq(i).contains(x)
    }
}

/// Membership in every term gives membership in the sequence intersection.
theorem seq_intersection_contains_of_forall[T](seq: Nat -> Set[T], x: T) {
    (forall(i: Nat) { seq(i).contains(x) }) implies seq_intersection(seq).contains(x)
} by {
    if forall(i: Nat) { seq(i).contains(x) } {
        seq_intersection_contains_eq(seq, x)
        seq_intersection(seq).contains(x)
    }
}

/// If a sequence intersection is empty, each element is absent from some term.
theorem seq_intersection_empty_imp_exists_not_contains[T](seq: Nat -> Set[T], x: T) {
    seq_intersection(seq).is_empty implies exists(i: Nat) {
        not seq(i).contains(x)
    }
} by {
    if seq_intersection(seq).is_empty {
        not seq_intersection(seq).contains(x)
        exists(i: Nat) {
            not seq(i).contains(x)
        }
    }
}

/// Pointwise inclusion of sequences preserves sequence unions.
theorem seq_union_monotone[T](a: Nat -> Set[T], b: Nat -> Set[T]) {
    (forall(i: Nat) { a(i).subset(b(i)) }) implies seq_union(a).subset(seq_union(b))
} by {
    if forall(i: Nat) { a(i).subset(b(i)) } {
        indexed_union_monotone(a, b)
        indexed_union(a).subset(indexed_union(b))
        indexed_union(a) = union_family(a)
        indexed_union(b) = union_family(b)
        seq_union(a) = union_family(a)
        seq_union(b) = union_family(b)
        seq_union(a).subset(seq_union(b))
    }
}

/// Pointwise inclusion of sequences preserves sequence intersections.
theorem seq_intersection_monotone[T](a: Nat -> Set[T], b: Nat -> Set[T]) {
    (forall(i: Nat) { a(i).subset(b(i)) }) implies
    seq_intersection(a).subset(seq_intersection(b))
} by {
    if forall(i: Nat) { a(i).subset(b(i)) } {
        indexed_intersection_monotone(a, b)
        indexed_intersection(a).subset(indexed_intersection(b))
        indexed_intersection(a) = intersection_family(a)
        indexed_intersection(b) = intersection_family(b)
        seq_intersection(a) = intersection_family(a)
        seq_intersection(b) = intersection_family(b)
        seq_intersection(a).subset(seq_intersection(b))
    }
}

/// A sequence union of empty sets is empty.
theorem seq_union_empty_of_forall_empty[T](seq: Nat -> Set[T]) {
    (forall(i: Nat) { seq(i).is_empty }) implies seq_union(seq) = Set[T].empty_set
} by {
    if forall(i: Nat) { seq(i).is_empty } {
        indexed_union_empty_of_forall_empty(seq)
        indexed_union(seq) = Set[T].empty_set
        seq_union(seq) = union_family(seq)
        indexed_union(seq) = union_family(seq)
        seq_union(seq) = Set[T].empty_set
    }
}

/// If a sequence union is empty, every term is empty.
theorem seq_union_empty_imp_forall_empty[T](seq: Nat -> Set[T]) {
    seq_union(seq) = Set[T].empty_set implies forall(i: Nat) {
        seq(i).is_empty
    }
} by {
    if seq_union(seq) = Set[T].empty_set {
        seq_union(seq) = union_family(seq)
        indexed_union(seq) = union_family(seq)
        indexed_union(seq) = Set[T].empty_set
        indexed_union_empty_imp_forall_empty(seq)
        forall(i: Nat) {
            seq(i).is_empty
        }
    }
}

/// A sequence intersection of universal sets is universal.
theorem seq_intersection_universal_of_forall_universal[T](seq: Nat -> Set[T]) {
    (forall(i: Nat) { seq(i) = Set[T].universal_set }) implies
    seq_intersection(seq) = Set[T].universal_set
} by {
    if forall(i: Nat) { seq(i) = Set[T].universal_set } {
        indexed_intersection_universal_of_forall_universal(seq)
        indexed_intersection(seq) = Set[T].universal_set
        seq_intersection(seq) = intersection_family(seq)
        indexed_intersection(seq) = intersection_family(seq)
        seq_intersection(seq) = Set[T].universal_set
    }
}

/// If a sequence intersection is universal, every term is universal.
theorem seq_intersection_universal_imp_forall_universal[T](seq: Nat -> Set[T]) {
    seq_intersection(seq) = Set[T].universal_set implies forall(i: Nat) {
        seq(i) = Set[T].universal_set
    }
} by {
    if seq_intersection(seq) = Set[T].universal_set {
        seq_intersection(seq) = intersection_family(seq)
        indexed_intersection(seq) = intersection_family(seq)
        indexed_intersection(seq) = Set[T].universal_set
        indexed_intersection_universal_imp_forall_universal(seq)
        forall(i: Nat) {
            seq(i) = Set[T].universal_set
        }
    }
}

theorem decreasing_seq_transitive[T](s: Nat -> Set[T], j: Nat, k: Nat) {
    is_decreasing(s) and j < k implies s(j).superset(s(k))
} by {
    let d: Nat satisfy {
        j + d = k and d != Nat.0
    }

    define f(dist: Nat) -> Bool {
        forall(i: Nat) {
            is_decreasing(s) implies s(i).superset(s(i + dist))
        }
    }

    forall(i: Nat) {
        i + Nat.0 = i
        s(i).subset(s(i))
        s(i).superset(s(i + Nat.0))
        is_decreasing(s) implies s(i).superset(s(i + Nat.0))
    }

    f(Nat.0)

    forall(dist: Nat) {
        if f(dist) {
            forall(i: Nat) {
                if is_decreasing(s) {
                    s(i + dist.suc).subset(s(i + dist))
                    s(i).superset(s(i + dist))
                    s(i + dist).subset(s(i))
                    s(i + dist.suc).subset(s(i))
                    s(i).superset(s(i + dist.suc))
                }
            }
            forall(i: Nat) {
                is_decreasing(s) implies s(i).superset(s(i + dist.suc))
            }
            exists(k0: Nat) {
                is_decreasing(s) and not s(k0).superset(s(k0 + dist.suc))
            } or f(dist.suc)
            f(dist.suc)
        }
    }

    f(d)
}

/// A decreasing sequence is antitone with respect to non-strict comparison.
theorem decreasing_seq_subset_of_le[T](s: Nat -> Set[T], j: Nat, k: Nat) {
    is_decreasing(s) and j <= k implies s(k).subset(s(j))
} by {
    if is_decreasing(s) and j <= k {
        if j = k {
            s(k).subset(s(j))
        } else {
            j < k
            decreasing_seq_transitive(s, j, k)
            s(j).superset(s(k))
            s(k).subset(s(j))
        }
    }
}

/// Later terms of a decreasing natural-number set family inherit lower bounds.
theorem decreasing_seq_lower_bound_of_le(s: Nat -> Set[Nat], j: Nat, k: Nat,
    bound: Nat) {
    is_decreasing(s) and j <= k and set_lower_bound(s(j), bound)
    implies set_lower_bound(s(k), bound)
} by {
    if is_decreasing(s) and j <= k and set_lower_bound(s(j), bound) {
        decreasing_seq_subset_of_le(s, j, k)
        s(k).subset(s(j))
        set_lower_bound_of_subset(s(k), s(j), bound)
        set_lower_bound(s(k), bound)
    }
}

/// Every term after an eventually bounded decreasing family remains bounded below.
theorem decreasing_seq_eventual_lower_bound(s: Nat -> Set[Nat], big_n: Nat,
    bound: Nat) {
    is_decreasing(s) and set_lower_bound(s(big_n), bound) implies forall(i: Nat) {
        big_n <= i implies set_lower_bound(s(i), bound)
    }
} by {
    if is_decreasing(s) and set_lower_bound(s(big_n), bound) {
        forall(i: Nat) {
            if big_n <= i {
                decreasing_seq_lower_bound_of_le(s, big_n, i, bound)
                set_lower_bound(s(i), bound)
            }
        }
    }
}

/// A decreasing sequence of natural-number sets with empty intersection is eventually above any bound.
theorem decreasing_empty_intersection_eventually_bounded(s: Nat -> Set[Nat], tail_n: Nat) {
    is_decreasing(s) and seq_intersection(s).is_empty
    implies
    exists(big_n: Nat) {
        set_lower_bound(s(big_n), tail_n)
    }
} by {
    define p(n: Nat) -> Bool {
        is_decreasing(s) and seq_intersection(s).is_empty
        implies
        exists(big_n: Nat) {
            set_lower_bound(s(big_n), n)
        }
    }

    forall(big_n: Nat) {
        forall(k: Nat) {
            s(big_n).contains(k) implies k >= Nat.0
        }
        set_lower_bound(s(big_n), Nat.0)
    }
    p(Nat.0)

    forall(n: Nat) {
        if p(n) and is_decreasing(s) and seq_intersection(s).is_empty {
            let big_n_prev: Nat satisfy {
                set_lower_bound(s(big_n_prev), n)
            }

            seq_intersection_empty_imp_exists_not_contains(s, n)
            let i_n: Nat satisfy {
                not s(i_n).contains(n)
            }

            if big_n_prev <= i_n {
                decreasing_seq_subset_of_le(s, big_n_prev, i_n)
                s(i_n).subset(s(big_n_prev))
                s(big_n_prev).superset(s(i_n))

                forall(k: Nat) {
                    if s(i_n).contains(k) {
                        s(i_n).subset(s(big_n_prev)) = forall(x: Nat) {
                            s(i_n).contains(x) implies s(big_n_prev).contains(x)
                        }
                        s(i_n).contains(k) implies s(big_n_prev).contains(k)
                        s(big_n_prev).contains(k)
                        set_lower_bound(s(big_n_prev), n)
                        set_lower_bound(s(big_n_prev), n) = forall(x: Nat) {
                            s(big_n_prev).contains(x) implies x >= n
                        }
                        s(big_n_prev).contains(k) implies k >= n
                        k >= n

                        not s(i_n).contains(n)
                        if k = n {
                            s(i_n).contains(n)
                        }
                        k != n
                        k >= n.suc
                    }
                }
                set_lower_bound(s(i_n), n.suc)

                exists(big_n: Nat) {
                    big_n = i_n and set_lower_bound(s(big_n), n.suc)
                }
                p(n.suc)
            } else {
                i_n <= big_n_prev
                decreasing_seq_subset_of_le(s, i_n, big_n_prev)
                s(big_n_prev).subset(s(i_n))
                s(i_n).superset(s(big_n_prev))

                forall(k: Nat) {
                    if s(big_n_prev).contains(k) {
                        s(big_n_prev).subset(s(i_n)) = forall(x: Nat) {
                            s(big_n_prev).contains(x) implies s(i_n).contains(x)
                        }
                        s(big_n_prev).contains(k) implies s(i_n).contains(k)
                        s(i_n).contains(k)

                        not s(i_n).contains(n)
                        if k = n {
                            s(i_n).contains(n)
                        }
                        k != n

                        set_lower_bound(s(big_n_prev), n)
                        set_lower_bound(s(big_n_prev), n) = forall(x: Nat) {
                            s(big_n_prev).contains(x) implies x >= n
                        }
                        s(big_n_prev).contains(k) implies k >= n
                        k >= n
                        k >= n.suc
                    }
                }
                set_lower_bound(s(big_n_prev), n.suc)

                exists(big_n: Nat) {
                    big_n = big_n_prev and set_lower_bound(s(big_n), n.suc)
                }
                p(n.suc)
            }
        }
    }

    forall(n: Nat) {
        p(n)
    }
    p(tail_n)
}

theorem seq_contains_intersection[T](s: Nat -> Set[T], i: Nat) {
    s(i).superset(seq_intersection(s))
} by {
    forall(x: T) {
        if seq_intersection(s).contains(x) {
            s(i).contains(x)
        }
    }
}

theorem increasing_seq_transitive[T](s: Nat -> Set[T], j: Nat, k: Nat) {
    is_increasing(s) and j < k implies s(j).subset(s(k))
} by {
    let d: Nat satisfy {
        j + d = k and d != Nat.0
    }

    define f(dist: Nat) -> Bool {
        forall(i: Nat) {
            is_increasing(s) implies s(i).subset(s(i + dist))
        }
    }

    forall(i: Nat) {
        i + Nat.0 = i
        s(i).subset(s(i))
        is_increasing(s) implies s(i).subset(s(i + Nat.0))
    }

    f(Nat.0)

    forall(dist: Nat) {
        if f(dist) {
            forall(i: Nat) {
                if is_increasing(s) {
                    s(i + dist).subset(s(i + dist.suc))
                    s(i).subset(s(i + dist))
                    s(i).subset(s(i + dist.suc))
                }
            }
            forall(i: Nat) {
                is_increasing(s) implies s(i).subset(s(i + dist.suc))
            }
            exists(k0: Nat) {
                is_increasing(s) and not s(k0).subset(s(k0 + dist.suc))
            } or f(dist.suc)
            f(dist.suc)
        }
    }

    f(d)
}

/// An increasing sequence is monotone with respect to non-strict comparison.
theorem increasing_seq_subset_of_le[T](s: Nat -> Set[T], j: Nat, k: Nat) {
    is_increasing(s) and j <= k implies s(j).subset(s(k))
} by {
    if is_increasing(s) and j <= k {
        if j = k {
            s(j).subset(s(k))
        } else {
            j < k
            increasing_seq_transitive(s, j, k)
            s(j).subset(s(k))
        }
    }
}

theorem seq_union_contains[T](s: Nat -> Set[T], i: Nat) {
    seq_union(s).superset(s(i))
} by {
    forall(x: T) {
        if s(i).contains(x) {
            seq_union(s).contains(x)
        }
    }
}

/// The range-indexed union over `n.suc` decomposes into the preceding range and the final term.
theorem range_indexed_union_suc_eq_union[K](n: Nat, family: Nat -> Set[K]) {
    range_indexed_union(n.suc, family) = range_indexed_union(n, family).union(family(n))
} by {
    let u = range_indexed_union(n.suc, family)
    let v = range_indexed_union(n, family).union(family(n))
    forall(x: K) {
        if u.contains(x) {
            range_indexed_union_contains_witness(n.suc, family, x)
            let i: Nat satisfy {
                i < n.suc and family(i).contains(x)
            }
            if i < n {
                range_indexed_union_contains_of_lt(n, family, i, x)
                range_indexed_union(n, family).contains(x)
                union_contains_eq(range_indexed_union(n, family), family(n), x)
                v.contains(x)
            } else {
                i = n
                family(n).contains(x)
                union_contains_eq(range_indexed_union(n, family), family(n), x)
                v.contains(x)
            }
        }
    }
    u.subset(v)
    forall(x: K) {
        if v.contains(x) {
            elem_in_union(range_indexed_union(n, family), family(n), x)
            range_indexed_union(n, family).contains(x) or family(n).contains(x)
            union_contains_eq(range_indexed_union(n, family), family(n), x)
            if range_indexed_union(n, family).contains(x) {
                range_indexed_union_contains_witness(n, family, x)
                let i: Nat satisfy {
                    i < n and family(i).contains(x)
                }
                i < n.suc
                range_indexed_union_contains_of_lt(n.suc, family, i, x)
                u.contains(x)
            }
            if family(n).contains(x) {
                n < n.suc
                range_indexed_union_contains_of_lt(n.suc, family, n, x)
                u.contains(x)
            }
            if not u.contains(x) {
                not range_indexed_union(n, family).contains(x)
                not family(n).contains(x)
                false
            }
            u.contains(x)
        }
    }
    v.subset(u)
    double_inclusion(u, v)
}

/// The range-indexed intersection over `n.suc` decomposes into the preceding range and the final term.
theorem range_indexed_intersection_suc_eq_intersection[K](n: Nat, family: Nat -> Set[K]) {
    range_indexed_intersection(n.suc, family) =
    range_indexed_intersection(n, family).intersection(family(n))
} by {
    let u = range_indexed_intersection(n.suc, family)
    let v = range_indexed_intersection(n, family).intersection(family(n))
    forall(x: K) {
        if u.contains(x) {
            forall(i: Nat) {
                if i < n {
                    i < n.suc
                    range_indexed_intersection_contains_at(n.suc, family, i, x)
                    family(i).contains(x)
                }
            }
            range_indexed_intersection_contains_of_forall(n, family, x)
            range_indexed_intersection(n, family).contains(x)
            n < n.suc
            range_indexed_intersection_contains_at(n.suc, family, n, x)
            family(n).contains(x)
            intersection_contains_eq(range_indexed_intersection(n, family), family(n), x)
            v.contains(x)
        }
        if v.contains(x) {
            intersection_contains_eq(range_indexed_intersection(n, family), family(n), x)
            range_indexed_intersection(n, family).contains(x)
            family(n).contains(x)
            forall(i: Nat) {
                if i < n.suc {
                    if i < n {
                        range_indexed_intersection_contains_at(n, family, i, x)
                        family(i).contains(x)
                    } else {
                        i = n
                        family(i).contains(x)
                    }
                }
            }
            range_indexed_intersection_contains_of_forall(n.suc, family, x)
            u.contains(x)
        }
        u.contains(x) = v.contains(x)
    }
    set_ext(u, v)
}

/// The finite union of an increasing family through `n` is contained in its final term.
theorem range_indexed_union_suc_subset_of_increasing[K](n: Nat, family: Nat -> Set[K]) {
    is_increasing(family) implies range_indexed_union(n.suc, family).subset(family(n))
} by {
    if is_increasing(family) {
        forall(i: Nat) {
            if i < n.suc {
                i <= n
                increasing_seq_subset_of_le(family, i, n)
                family(i).subset(family(n))
            }
        }
        range_indexed_union_subset_of_family_subset(n.suc, family, family(n))
        range_indexed_union(n.suc, family).subset(family(n))
    }
}

/// The final term belongs to the finite union through it.
theorem family_subset_range_indexed_union_suc[K](n: Nat, family: Nat -> Set[K]) {
    family(n).subset(range_indexed_union(n.suc, family))
} by {
    forall(x: K) {
        if family(n).contains(x) {
            n < n.suc
            range_indexed_union_contains_of_lt(n.suc, family, n, x)
            range_indexed_union(n.suc, family).contains(x)
        }
    }
}

/// The finite union of an increasing family through `n` is its final term.
theorem range_indexed_union_suc_of_increasing[K](n: Nat, family: Nat -> Set[K]) {
    is_increasing(family) implies range_indexed_union(n.suc, family) = family(n)
} by {
    if is_increasing(family) {
        range_indexed_union_suc_subset_of_increasing(n, family)
        family_subset_range_indexed_union_suc(n, family)
        double_inclusion(range_indexed_union(n.suc, family), family(n))
    }
}

/// The final term of a finite decreasing family is contained in the finite intersection through it.
theorem decreasing_subset_range_indexed_intersection_suc[K](n: Nat, family: Nat -> Set[K]) {
    is_decreasing(family) implies family(n).subset(range_indexed_intersection(n.suc, family))
} by {
    if is_decreasing(family) {
        forall(i: Nat) {
            if i < n.suc {
                i <= n
                decreasing_seq_subset_of_le(family, i, n)
                family(n).subset(family(i))
            }
        }
        subset_range_indexed_intersection_of_subset_family(n.suc, family, family(n))
        family(n).subset(range_indexed_intersection(n.suc, family))
    }
}

/// The finite intersection through `n` is contained in its final term.
theorem range_indexed_intersection_suc_subset_family[K](n: Nat, family: Nat -> Set[K]) {
    range_indexed_intersection(n.suc, family).subset(family(n))
} by {
    forall(x: K) {
        if range_indexed_intersection(n.suc, family).contains(x) {
            n < n.suc
            range_indexed_intersection_contains_at(n.suc, family, n, x)
            family(n).contains(x)
        }
    }
}

/// The finite intersection of a decreasing family through `n` is its final term.
theorem range_indexed_intersection_suc_of_decreasing[K](n: Nat, family: Nat -> Set[K]) {
    is_decreasing(family) implies range_indexed_intersection(n.suc, family) = family(n)
} by {
    if is_decreasing(family) {
        decreasing_subset_range_indexed_intersection_suc(n, family)
        range_indexed_intersection_suc_subset_family(n, family)
        double_inclusion(range_indexed_intersection(n.suc, family), family(n))
    }
}

/// A range-indexed union is contained in the full sequence union.
theorem range_indexed_union_subset_seq_union[K](n: Nat, family: Nat -> Set[K]) {
    range_indexed_union(n, family).subset(seq_union(family))
} by {
    forall(x: K) {
        if range_indexed_union(n, family).contains(x) {
            range_indexed_union_contains_witness(n, family, x)
            let i: Nat satisfy {
                i < n and family(i).contains(x)
            }
            seq_union_contains_of_contains(family, i, x)
            seq_union(family).contains(x)
        }
    }
}

/// The full sequence intersection is contained in any range-indexed intersection.
theorem seq_intersection_subset_range_indexed_intersection[K](n: Nat, family: Nat -> Set[K]) {
    seq_intersection(family).subset(range_indexed_intersection(n, family))
} by {
    forall(x: K) {
        if seq_intersection(family).contains(x) {
            forall(i: Nat) {
                if i < n {
                    seq_intersection_contains_at(family, i, x)
                    family(i).contains(x)
                }
            }
            range_indexed_intersection_contains_of_forall(n, family, x)
            range_indexed_intersection(n, family).contains(x)
        }
    }
}

/// The pointwise image of a sequence has union equal to the image of the union.
theorem seq_union_set_image[T, U](seq: Nat -> Set[T], f: T -> U) {
    seq_union(set_image_seq(seq, f)) = set_image(seq_union(seq), f)
} by {
    let u = seq_union(set_image_seq(seq, f))
    let v = set_image(seq_union(seq), f)
    let u_simp = union_family(set_image_seq(seq, f))
    let su = seq_union(seq)
    let su_simp = union_family(seq)

    forall(y: U) {
        if u.contains(y) {
            u_simp.contains(y)
            let i: Nat satisfy {
                set_image_seq(seq, f, i).contains(y)
            }
            set_image_seq(seq, f, i) = set_image(seq(i), f)
            set_image_contains_witness(seq(i), f, y)
            let x: T satisfy {
                seq(i).contains(x) and y = f(x)
            }
            seq_union_contains(seq, i)
            su.superset(seq(i))
            su.contains(x)
            maps_into_set_image(su, f, x)
            v.contains(f(x))
            v.contains(y)
        }

        if v.contains(y) {
            set_image_contains_witness(su, f, y)
            let x: T satisfy {
                su.contains(x) and y = f(x)
            }
            su = su_simp
            su_simp.contains(x)
            let i: Nat satisfy {
                seq(i).contains(x)
            }
            set_image_seq(seq, f, i) = set_image(seq(i), f)
            maps_into_set_image(seq(i), f, x)
            set_image_seq(seq, f, i).contains(f(x))
            u_simp.contains(f(x))
            u.contains(f(x))
            u.contains(y)
        }

        u.contains(y) = v.contains(y)
    }

    set_ext(u, v)
}

/// The union of the natural initial segments is the universal set.
theorem seq_union_nat_lt_set_universal {
    seq_union(nat_lt_set) = Set[Nat].universal_set
} by {
    let u = seq_union(nat_lt_set)
    let v = Set[Nat].universal_set

    forall(n: Nat) {
        if u.contains(n) {
            universal_set_contains_eq[Nat](n)
            v.contains(n)
        }

        if v.contains(n) {
            universal_set_contains_eq[Nat](n)
            n < n.suc
            nat_lt_set_contains_eq(n.suc, n)
            nat_lt_set(n.suc).contains(n)
            seq_union_contains(nat_lt_set, n.suc)
            u.superset(nat_lt_set(n.suc))
            u.contains(n)
        }

        u.contains(n) = v.contains(n)
    }

    set_ext(u, v)
}

/// The image of the initial segment `{0, 1, ..., k - 1}` under a function.
define image_of_range[T](p: Nat -> T, k: Nat) -> Set[T] {
    set_image(nat_lt_set(k), p)
}

/// The image of a finite initial segment agrees with the generic image construction.
theorem image_of_range_eq_set_image[T](p: Nat -> T, k: Nat) {
    image_of_range(p, k) = set_image(nat_lt_set(k), p)
}

/// The image sequence of initial segments is increasing.
theorem image_of_range_increasing[T](p: Nat -> T) {
    is_increasing(image_of_range(p))
} by {
    forall(k: Nat) {
        image_of_range_eq_set_image(p, k)
        image_of_range_eq_set_image(p, k.suc)
        nat_lt_set_subset_suc(k)
        set_image_monotone(nat_lt_set(k), nat_lt_set(k.suc), p)
        image_of_range(p, k).subset(image_of_range(p, k.suc))
    }
    is_increasing(image_of_range(p))
}

/// The union of the image sequence is the image of the universal set.
theorem image_of_range_union_eq_universal_image[T](p: Nat -> T) {
    seq_union(image_of_range(p)) = set_image(Set[Nat].universal_set, p)
} by {
    forall(k: Nat) {
        image_of_range_eq_set_image(p, k)
        set_image_seq(nat_lt_set, p, k) = set_image(nat_lt_set(k), p)
        image_of_range(p, k) = set_image_seq(nat_lt_set, p, k)
    }
    function_extensionality(image_of_range(p), set_image_seq(nat_lt_set, p))
    seq_union(image_of_range(p)) = seq_union(set_image_seq(nat_lt_set, p))
    seq_union_set_image(nat_lt_set, p)
    seq_union(image_of_range(p)) = set_image(seq_union(nat_lt_set), p)
    seq_union_nat_lt_set_universal
    seq_union(image_of_range(p)) = set_image(Set[Nat].universal_set, p)
}

/// A surjective function has universal union over its initial-segment images.
theorem image_of_range_union_universal[T](p: Nat -> T) {
    is_surjective_fn(p)
    implies
    seq_union(image_of_range(p)) = Set[T].universal_set
} by {
    if is_surjective_fn(p) {
        image_of_range_union_eq_universal_image(p)
        seq_union(image_of_range(p)) = set_image(Set[Nat].universal_set, p)
        set_image_universal_of_surjective(p)
        set_image(Set[Nat].universal_set, p) = Set[T].universal_set
        seq_union(image_of_range(p)) = Set[T].universal_set
    }
}

/// The image of an initial segment is disjoint from the next value of an injective function.
theorem image_of_range_disjoint_from_next[T](p: Nat -> T, m: Nat) {
    is_injective_fn(p) implies image_of_range(p, m).is_disjoint(Set[T].singleton(p(m)))
} by {
    if is_injective_fn(p) {
        forall(n: T) {
            if image_of_range(p, m).contains(n) and Set[T].singleton(p(m)).contains(n) {
                image_of_range_eq_set_image(p, m)
                set_image_contains_witness(nat_lt_set(m), p, n)
                let i: Nat satisfy {
                    nat_lt_set(m).contains(i) and n = p(i)
                }
                p(m) = n
                p(i) = p(m)
                is_injective_fn(p)
                i = m
                nat_lt_set_contains_eq(m, i)
                i < m
                false
            }
        }
    }
}

/// The image of a successor initial segment is the previous image together with the endpoint.
theorem image_of_range_suc_eq_union_singleton[T](p: Nat -> T, m: Nat) {
    image_of_range(p, m.suc) = image_of_range(p, m).union(Set[T].singleton(p(m)))
} by {
    image_of_range_eq_set_image(p, m.suc)
    nat_lt_set_suc_eq_union(m)
    set_image_union(nat_lt_set(m), Set[Nat].singleton(m), p)
    set_image_singleton(m, p)
    image_of_range_eq_set_image(p, m)
    image_of_range(p, m.suc) = image_of_range(p, m).union(Set[T].singleton(p(m)))
}

theorem seq_union_complement_eq_intersection_complement[T](s: Nat -> Set[T]) {
    seq_union(seq_complement(s)) = (seq_intersection(s)).c
} by {
    let u = seq_union(seq_complement(s))
    let v = (seq_intersection(s)).c
    let sc = seq_complement(s)
    let u_simp = union_family(sc)

    forall(x: T) {
        if u.contains(x) {
            u_simp.contains(x)
            let (i: Nat) satisfy {
                sc(i).contains(x)
            }
            sc(i) = s(i).c
            s(i).c.contains(x)
            not s(i).contains(x)
            not seq_intersection(s).contains(x)
            v.contains(x)
        }

        if v.contains(x) {
            not seq_intersection(s).contains(x)
            let si = seq_intersection(s)
            let si_simp = intersection_family(s)
            si = si_simp
            not si_simp.contains(x)
            let (i: Nat) satisfy {
                not s(i).contains(x)
            }
            s(i).c.contains(x)
            sc(i) = s(i).c
            sc(i).contains(x)
            u_simp.contains(x)
            u.contains(x)
        }

        u.contains(x) = v.contains(x)
    }

    set_ext(u, v)
}

theorem seq_intersection_complement_eq_union_complement[T](s: Nat -> Set[T]) {
    seq_intersection(seq_complement(s)) = (seq_union(s)).c
} by {
    let u = seq_intersection(seq_complement(s))
    let v = (seq_union(s)).c
    let sc = seq_complement(s)
    let u_simp = intersection_family(sc)

    forall(x: T) {
        if u.contains(x) {
            u_simp.contains(x)
            forall(i: Nat) {
                sc(i).contains(x)
                sc(i) = s(i).c
                s(i).c.contains(x)
                not s(i).contains(x)
            }
            let su = seq_union(s)
            let su_simp = union_family(s)
            su = su_simp
            if su_simp.contains(x) {
                let (i: Nat) satisfy {
                    s(i).contains(x)
                }
                s(i).contains(x)
                not s(i).contains(x)
                false
            }
            not su_simp.contains(x)
            not seq_union(s).contains(x)
            v.contains(x)
        }

        if v.contains(x) {
            not seq_union(s).contains(x)
            forall(i: Nat) {
                not s(i).contains(x)
                s(i).c.contains(x)
                sc(i) = s(i).c
                sc(i).contains(x)
            }
            and_family(sc, x)
            u_simp.contains(x)
            u.contains(x)
        }

        u.contains(x) = v.contains(x)
    }

    set_ext(u, v)
}

/// If a sequence union is universal, the intersection of the complements is empty.
theorem seq_union_universal_imp_complement_intersection_empty[T](s: Nat -> Set[T]) {
    seq_union(s) = Set[T].universal_set implies seq_intersection(seq_complement(s)).is_empty
} by {
    if seq_union(s) = Set[T].universal_set {
        seq_intersection_complement_eq_union_complement(s)
        seq_intersection(seq_complement(s)) = (seq_union(s)).c
        (seq_union(s)).c = (Set[T].universal_set).c
        universal_set_compl_is_empty[T]
        (Set[T].universal_set).c = Set[T].empty_set
        seq_intersection(seq_complement(s)) = Set[T].empty_set
        empty_set_is_empty[T]
        seq_intersection(seq_complement(s)).is_empty
    }
}

/// If a sequence intersection is empty, the union of the complements is universal.
theorem seq_intersection_empty_imp_complement_union_universal[T](s: Nat -> Set[T]) {
    seq_intersection(s).is_empty implies seq_union(seq_complement(s)) = Set[T].universal_set
} by {
    if seq_intersection(s).is_empty {
        set_eq_empty_of_is_empty(seq_intersection(s))
        seq_intersection(s) = Set[T].empty_set
        seq_union_complement_eq_intersection_complement(s)
        seq_union(seq_complement(s)) = (seq_intersection(s)).c
        (seq_intersection(s)).c = (Set[T].empty_set).c
        empty_set_compl_is_universal[T]
        (Set[T].empty_set).c = Set[T].universal_set
        seq_union(seq_complement(s)) = Set[T].universal_set
    }
}

/// If the complement sequence has empty intersection, the original sequence has universal union.
theorem seq_complement_intersection_empty_imp_union_universal[T](s: Nat -> Set[T]) {
    seq_intersection(seq_complement(s)).is_empty implies
    seq_union(s) = Set[T].universal_set
} by {
    if seq_intersection(seq_complement(s)).is_empty {
        set_eq_empty_of_is_empty(seq_intersection(seq_complement(s)))
        seq_intersection(seq_complement(s)) = Set[T].empty_set
        seq_intersection_complement_eq_union_complement(s)
        seq_intersection(seq_complement(s)) = (seq_union(s)).c
        (seq_union(s)).c = Set[T].empty_set
        (seq_union(s)).c.c = (Set[T].empty_set).c
        compl_of_compl_is_self(seq_union(s))
        empty_set_compl_is_universal[T]
        (Set[T].empty_set).c = Set[T].universal_set
        seq_union(s) = Set[T].universal_set
    }
}

/// If a sequence union is universal, the union of complement complements is universal.
theorem seq_union_universal_imp_complement_complement_union_universal[T](s: Nat -> Set[T]) {
    seq_union(s) = Set[T].universal_set implies
    seq_union(seq_complement(seq_complement(s))) = Set[T].universal_set
} by {
    if seq_union(s) = Set[T].universal_set {
        seq_complement_complement(s)
        seq_complement(seq_complement(s)) = s
        seq_union(seq_complement(seq_complement(s))) = seq_union(s)
        seq_union(seq_complement(seq_complement(s))) = Set[T].universal_set
    }
}

// Proving indexed union and intersection definitions are consistent
// with family definitions
/// A type with exactly two values.
inductive TwoType {
    /// The first value.
    first
    /// The second value.
    second
}

// Creates a function f: TwoType -> Set[K]
define fun_of_two[K](a: Set[K], b: Set[K], x: TwoType) -> Set[K] {
    match x {
        TwoType.first {
            a
        }
        TwoType.second {
            b
        }
    }
}

// Defines union in the case of two sets using family definition
define family_union_of_two[K](a: Set[K], b: Set[K]) -> Set[K] {
    union_family(fun_of_two(a, b))
}

theorem union_is_family_union_of_two[K](a: Set[K], b: Set[K]) {
    a.union(b) = family_union_of_two(a, b)
} by {
    let u = a.union(b)
    let v = family_union_of_two(a, b)
    let f_two = fun_of_two(a, b)
    // v_simp unrolls definitions, which helps the prover
    let v_simp = union_family(f_two)

    // Helps with definitions
    f_two(TwoType.first) = a
    f_two(TwoType.second) = b

    // First prove that u \subseteq v
    forall(x: K) {
        // Somehow this helps the prover though it only really unrolls definitions
        if a.contains(x) {
            f_two(TwoType.first).contains(x)
            or_family(f_two, x)
            v_simp.contains(x)
        }
        a.contains(x) implies v_simp.contains(x)
        if b.contains(x) {
            f_two(TwoType.second).contains(x)
            or_family(f_two, x)
            v_simp.contains(x)
        }
        b.contains(x) implies v_simp.contains(x)

        if u.contains(x) {
            elem_in_union(a, b, x)
            if a.contains(x) {
                v_simp.contains(x)
            } else {
                b.contains(x)
                v_simp.contains(x)
            }
            v.contains(x)
        }
        u.contains(x) implies v.contains(x)
    }

    // Now prove that v \subseteq u
    forall(x: K) {
        if v.contains(x) {
            // Definition of v.contains(x), unrolled
            v_simp.contains(x)

            let (t: TwoType) satisfy {
                f_two(t).contains(x)
            }

            match t {
                TwoType.first {
                    a.contains(x)
                    elem_in_union(a, b, x)
                    u.contains(x)
                }
                TwoType.second {
                    b.contains(x)
                    elem_in_union(a, b, x)
                    u.contains(x)
                }
            }

            u.contains(x)
        }
    }
    v.subset(u)
    u.subset(v)
}

theorem intersection_is_family_intersection_of_two[K](a: Set[K], b: Set[K]) {
    a.intersection(b) = intersection_family(fun_of_two(a, b))
} by {
    let u = a.intersection(b)
    let v = intersection_family(fun_of_two(a, b))
    let f_two = fun_of_two(a, b)

    f_two(TwoType.first) = a
    f_two(TwoType.second) = b

    // First prove that u \subseteq v
    forall(x: K) {
        if u.contains(x) {
            elem_in_intersection(a, b, x)
            a.contains(x)
            b.contains(x)

            forall(t: TwoType) {
                match t {
                    TwoType.first {
                        f_two(t).contains(x)
                    }
                    TwoType.second {
                        f_two(t).contains(x)
                    }
                }
            }
            and_family(f_two, x)
            v.contains(x)
        }

        u.contains(x) implies v.contains(x)
    }

    // Makes life a little easier for the prover

    u.subset(v)

    // Now prove that v \subseteq u
    forall(x: K) {
        if v.contains(x) {
            // Definitions unrolled
            f_two(TwoType.first).contains(x)
            f_two(TwoType.second).contains(x)

            u.contains(x)
        }
    }
    v.subset(u)
}

attributes List[K] {
    /// The set whose elements are the elements of the list.
    define as_set(self) -> Set[K] {
        Set[K].new(self.contains)
    }

    define contains_set(self, s: Set[K]) -> Bool {
        forall(x: K) {
            s.contains(x) implies self.contains(x)
        }
    }
}

/// Pointwise consequence of `contains_set`: a containing list contains every
/// member of the set.
theorem contains_set_at[K](l: List[K], s: Set[K], x: K) {
    l.contains_set(s) and s.contains(x) implies l.contains(x)
} by {
    if l.contains_set(s) and s.contains(x) {
        l.contains_set(s) = forall(y: K) {
            s.contains(y) implies l.contains(y)
        }
        s.contains(x) implies l.contains(x)
    }
}

/// Introduction lemma for `contains_set`: pointwise containment gives the
/// `contains_set` predicate.
theorem contains_set_intro[K](l: List[K], s: Set[K]) {
    (forall(x: K) { s.contains(x) implies l.contains(x) }) implies l.contains_set(s)
} by {
    if forall(x: K) { s.contains(x) implies l.contains(x) } {
        l.contains_set(s) = forall(y: K) {
            s.contains(y) implies l.contains(y)
        }
    }
}

/// The set associated to a list.
define list_set[K](items: List[K]) -> Set[K] {
    items.as_set
}

// Finite-set attributes and lemmas
attributes Set[K] {
    /// True if the set contains only finitely many elements.
    define is_finite(self) -> Bool {
        finite_constraint(self.contains)
    }

    /// True if the set contains infinitely many elements.
    define is_infinite(self) -> Bool {
        not self.is_finite
    }

    /// True if the cardinality is at most n.
    define cardinality_at_most(self, n: Nat) -> Bool {
        exists (superset: List[K]) {
            superset.contains_set(self) and superset.length <= n
        }
    }

    /// True if the cardinality equals n.
    define cardinality_is(self, n: Nat) -> Bool {
        exists (containing_list: List[K]) {
            containing_list.contains_set(self) and containing_list.filter(self.contains).unique.length = n
        }
    }
}

/// Membership in the set associated to a list is list membership.
theorem list_set_contains_eq[K](items: List[K], x: K) {
    list_set(items).contains(x) = items.contains(x)
}

/// The set associated to a list is finite.
theorem list_set_is_finite[K](items: List[K]) {
    list_set(items).is_finite
} by {
    list_contains_satisfies_finite_constraint(items)
    finite_constraint(items.contains)
    list_set(items).contains = items.contains
    finite_constraint(list_set(items).contains)
    list_set(items).is_finite
}

/// The set associated to a list has cardinality at most the length of the list.
theorem list_set_cardinality_at_most_length[K](items: List[K]) {
    list_set(items).cardinality_at_most(items.length)
} by {
    forall(x: K) {
        if list_set(items).contains(x) {
            list_set_contains_eq(items, x)
            items.contains(x)
        }
    }
    items.contains_set(list_set(items))
    items.length <= items.length
    list_set(items).cardinality_at_most(items.length)
}

/// Membership in the set associated to a filtered list is list membership together with the filter predicate.
theorem list_set_filter_contains_eq[K](items: List[K], f: K -> Bool, x: K) {
    list_set(items.filter(f)).contains(x) = (list_set(items).contains(x) and f(x))
} by {
    list_set_contains_eq(items.filter(f), x)
    list_set_contains_eq(items, x)
    filter_equivalent_to_and(items, f, x)
    list_set(items.filter(f)).contains(x) = items.filter(f).contains(x)
    items.filter(f).contains(x) = (items.contains(x) and f(x))
    list_set(items).contains(x) = items.contains(x)
    list_set(items.filter(f)).contains(x) = (list_set(items).contains(x) and f(x))
}

/// The set associated to a filtered list is contained in the set associated to the original list.
theorem list_set_filter_subset_list_set[K](items: List[K], f: K -> Bool) {
    list_set(items.filter(f)).subset(list_set(items))
} by {
    forall(x: K) {
        if list_set(items.filter(f)).contains(x) {
            list_set_filter_contains_eq(items, f, x)
            list_set(items).contains(x)
        }
    }
}

/// The set associated to a filtered list is contained in the filter predicate.
theorem list_set_filter_subset_predicate[K](items: List[K], f: K -> Bool) {
    list_set(items.filter(f)).subset(Set[K].new(f))
} by {
    forall(x: K) {
        if list_set(items.filter(f)).contains(x) {
            list_set_filter_contains_eq(items, f, x)
            f(x)
            Set[K].new(f).contains(x)
        }
    }
}

/// Filtering a list corresponds to intersecting its associated set with the predicate set.
theorem list_set_filter_eq_intersection[K](items: List[K], f: K -> Bool) {
    list_set(items.filter(f)) = list_set(items).intersection(Set[K].new(f))
} by {
    let u = list_set(items.filter(f))
    let v = list_set(items).intersection(Set[K].new(f))
    forall(x: K) {
        list_set_filter_contains_eq(items, f, x)
        intersection_contains_eq(list_set(items), Set[K].new(f), x)
        if u.contains(x) {
            list_set(items).contains(x)
            f(x)
            Set[K].new(f).contains(x)
            v.contains(x)
        }
        if v.contains(x) {
            list_set(items).contains(x)
            Set[K].new(f).contains(x)
            f(x)
            u.contains(x)
        }
        u.contains(x) = v.contains(x)
    }
    set_ext(u, v)
}

/// Filtering a list by a subset predicate extracts that subset.
theorem list_set_filter_extracts_subset[K](items: List[K], s: Set[K]) {
    s.subset(list_set(items)) implies list_set(items.filter(s.contains)) = s
} by {
    if s.subset(list_set(items)) {
        list_set_filter_eq_intersection(items, s.contains)
        forall(x: K) {
            Set[K].new(s.contains).contains(x) = s.contains(x)
        }
        set_ext(Set[K].new(s.contains), s)
        Set[K].new(s.contains) = s
        list_set(items.filter(s.contains)) = list_set(items).intersection(s)
        intersection_comm(list_set(items), s)
        list_set(items).intersection(s) = s.intersection(list_set(items))
        intersection_with_superset_is_self(s, list_set(items))
        s.intersection(list_set(items)) = s
        list_set(items.filter(s.contains)) = s
    }
}

/// A subset of a list-backed set is itself list-backed.
theorem list_set_subset_has_list[K](items: List[K], s: Set[K]) {
    s.subset(list_set(items)) implies exists(subitems: List[K]) {
        list_set(subitems) = s
    }
} by {
    if s.subset(list_set(items)) {
        let subitems = items.filter(s.contains)
        list_set_filter_extracts_subset(items, s)
        list_set(subitems) = s
        exists(result: List[K]) {
            list_set(result) = s
        }
    }
}

/// A unique list has cardinality equal to its length as a set.
theorem unique_list_set_cardinality_is_length[K](items: List[K]) {
    items.is_unique implies list_set(items).cardinality_is(items.length)
} by {
    if items.is_unique {
        forall(x: K) {
            if list_set(items).contains(x) {
                list_set_contains_eq(items, x)
                items.contains(x)
            }
        }
        items.contains_set(list_set(items))
        items.filter(items.contains) = items
        items.unique = items
        items.filter(list_set(items).contains) = items
        items.filter(list_set(items).contains).unique.length = items.length
        list_set(items).cardinality_is(items.length)
    }
}

theorem empty_set_cardinality_is_zero[T] {
    Set[T].empty_set.cardinality_is(Nat.0)
} by {
    List.nil[T].contains_set(Set[T].empty_set)
}

/// A finite set is not infinite.
theorem finite_set_not_infinite[T](s: Set[T]) {
    s.is_finite implies not s.is_infinite
}

/// An infinite set is not finite.
theorem infinite_set_not_finite[T](s: Set[T]) {
    s.is_infinite implies not s.is_finite
}

/// A set is infinite exactly when it is not finite.
theorem is_infinite_iff_not_finite[T](s: Set[T]) {
    s.is_infinite = not s.is_finite
}

/// A singleton set is finite.
theorem singleton_set_is_finite[T](a: T) {
    (Set[T].singleton(a)).is_finite
} by {
    let superset = List.singleton[T](a)

    forall(x: T) {
        if Set[T].singleton(a).contains(x) {
            singleton_contains_eq(a, x)
            a = x
            superset.contains(x)
        }
    }

    finite_constraint((Set[T].singleton(a)).contains)
    (Set[T].singleton(a)).is_finite
}

/// A singleton set has cardinality one.
theorem singleton_set_cardinality_is_one[T](a: T) {
    (Set[T].singleton(a)).cardinality_is(Nat.1)
} by {
    let items = List.cons(a, List.nil[T])
    not List.nil[T].contains(a)
    items.is_unique
    list_set(items).cardinality_is(items.length)
    items.length = List.nil[T].length.suc
    List.nil[T].length = Nat.0
    items.length = Nat.0.suc

    forall(x: T) {
        singleton_contains_eq(a, x)
        list_set_contains_eq(items, x)
        if (Set[T].singleton(a)).contains(x) {
            items.contains(x)
            list_set(items).contains(x)
        }
        if list_set(items).contains(x) {
            items.contains(x)
            if a != x {
                items.contains(x) = List.nil[T].contains(x)
                not List.nil[T].contains(x)
                not items.contains(x)
                false
            }
            (Set[T].singleton(a)).contains(x)
        }
        (Set[T].singleton(a)).contains(x) = list_set(items).contains(x)
    }
    set_ext(Set[T].singleton(a), list_set(items))
    (Set[T].singleton(a)).cardinality_is(items.length)
    Nat.1 = Nat.0.suc
    (Set[T].singleton(a)).cardinality_is(Nat.1)
}

/// Inserting an element into a finite set gives a finite set.
theorem insert_is_finite_of_finite[T](s: Set[T], item: T) {
    s.is_finite implies s.insert(item).is_finite
} by {
    if s.is_finite {
        finite_constraint(s.contains)
        functional_insert_satisfies_finite_constraint(s.contains, item)
        finite_constraint(functional_insert(s.contains, item))

        let superset: List[T] satisfy {
            forall(x: T) {
                functional_insert(s.contains, item, x) implies superset.contains(x)
            }
        }

        forall(x: T) {
            if s.insert(item).contains(x) {
                functional_insert(s.contains, item, x)
                superset.contains(x)
            }
        }

        finite_constraint(s.insert(item).contains)
        s.insert(item).is_finite
    }
}

/// Removing an element from a finite set gives a finite set.
theorem remove_is_finite_of_finite[T](s: Set[T], item: T) {
    s.is_finite implies s.remove(item).is_finite
} by {
    if s.is_finite {
        finite_constraint(s.contains)
        functional_remove_satisfies_finite_constraint(s.contains, item)
        finite_constraint(functional_remove(s.contains, item))

        let superset: List[T] satisfy {
            forall(x: T) {
                functional_remove(s.contains, item, x) implies superset.contains(x)
            }
        }

        forall(x: T) {
            if s.remove(item).contains(x) {
                functional_remove(s.contains, item, x)
                superset.contains(x)
            }
        }

        finite_constraint(s.remove(item).contains)
        s.remove(item).is_finite
    }
}

theorem union_is_finite_of_finite[K](a: Set[K], b: Set[K]) {
    a.is_finite and b.is_finite implies (a.union(b)).is_finite
} by {
    finite_constraint(a.contains)
    finite_constraint(b.contains)

    let superset_a: List[K] satisfy {
        forall(x: K) {
            a.contains(x) implies superset_a.contains(x)
        }
    }
    let superset_b: List[K] satisfy {
        forall(x: K) {
            b.contains(x) implies superset_b.contains(x)
        }
    }

    let superset_union = superset_a + superset_b

    forall(x: K) {
        if a.union(b).contains(x) {
            elem_in_union(a, b, x)
            if a.contains(x) {
                superset_a.contains(x)
                superset_union.contains(x)
            } else {
                b.contains(x)
                superset_b.contains(x)
                superset_union.contains(x)
            }
        }
    }

    exists(superset: List[K]) {
        forall(x: K) {
            a.union(b).contains(x) implies superset.contains(x)
        }
    }
    finite_constraint(a.union(b).contains)
    a.union(b).is_finite
}

theorem intersection_is_finite_of_finite[K](a: Set[K], b: Set[K]) {
    a.is_finite and b.is_finite implies (a.intersection(b)).is_finite
} by {
    if a.is_finite and b.is_finite {
        finite_constraint(a.contains)

        let superset_a: List[K] satisfy {
            forall(x: K) {
                a.contains(x) implies superset_a.contains(x)
            }
        }

        forall(x: K) {
            if a.intersection(b).contains(x) {
                elem_in_intersection(a, b, x)
                a.contains(x)
                superset_a.contains(x)
            }
        }

        finite_constraint(a.intersection(b).contains)
        a.intersection(b).is_finite
    }
}

theorem difference_is_finite_of_finite[K](a: Set[K], b: Set[K]) {
    a.is_finite implies (a.difference(b)).is_finite
} by {
    if a.is_finite {
        finite_constraint(a.contains)

        let superset_a: List[K] satisfy {
            forall(x: K) {
                a.contains(x) implies superset_a.contains(x)
            }
        }

        forall(x: K) {
            if a.difference(b).contains(x) {
                elem_in_difference(a, b, x)
                a.contains(x)
                superset_a.contains(x)
            }
        }

        finite_constraint(a.difference(b).contains)
        a.difference(b).is_finite
    }
}

/// A subset of a finite set is finite.
theorem subset_is_finite_of_finite[K](a: Set[K], b: Set[K]) {
    a.subset(b) and b.is_finite implies a.is_finite
} by {
    if a.subset(b) and b.is_finite {
        finite_constraint(b.contains)

        let superset_b: List[K] satisfy {
            forall(x: K) {
                b.contains(x) implies superset_b.contains(x)
            }
        }

        forall(x: K) {
            if a.contains(x) {
                b.contains(x)
                superset_b.contains(x)
            }
        }

        finite_constraint(a.contains)
        a.is_finite
    }
}

/// A range-indexed intersection is finite when one indexed member below the bound is finite.
theorem range_indexed_intersection_is_finite_of_member_finite[K](n: Nat,
    family: Nat -> Set[K], i: Nat) {
    i < n and family(i).is_finite implies range_indexed_intersection(n, family).is_finite
} by {
    if i < n and family(i).is_finite {
        forall(x: K) {
            if range_indexed_intersection(n, family).contains(x) {
                range_indexed_intersection_contains_at(n, family, i, x)
                family(i).contains(x)
            }
        }
        range_indexed_intersection(n, family).subset(family(i))
        subset_is_finite_of_finite(range_indexed_intersection(n, family), family(i))
        range_indexed_intersection(n, family).is_finite
    }
}

/// A superset of an infinite set is infinite.
theorem superset_is_infinite_of_infinite[K](a: Set[K], b: Set[K]) {
    a.subset(b) and a.is_infinite implies b.is_infinite
} by {
    if a.subset(b) and a.is_infinite {
        if b.is_finite {
            subset_is_finite_of_finite(a, b)
            a.is_finite
            false
        }
        b.is_infinite
    }
}

/// A subset whose superset is finite is not infinite.
theorem subset_not_infinite_of_finite[K](a: Set[K], b: Set[K]) {
    a.subset(b) and b.is_finite implies not a.is_infinite
} by {
    if a.subset(b) and b.is_finite {
        subset_is_finite_of_finite(a, b)
        a.is_finite
        not a.is_infinite
    }
}

/// The image of a finite set is finite.
theorem set_image_is_finite_of_finite[T, U](s: Set[T], f: T -> U) {
    s.is_finite implies set_image(s, f).is_finite
} by {
    if s.is_finite {
        finite_constraint(s.contains)

        let superset: List[T] satisfy {
            forall(x: T) {
                s.contains(x) implies superset.contains(x)
            }
        }

        let image_superset = superset.map(f)

        forall(y: U) {
            if set_image(s, f).contains(y) {
                set_image_contains_witness(s, f, y)
                let x: T satisfy {
                    s.contains(x) and y = f(x)
                }
                superset.contains_set(s) = forall(z: T) {
                    s.contains(z) implies superset.contains(z)
                }
                s.contains(x) implies superset.contains(x)
                superset.contains(x)
                map_contains_of_contains(superset, f, x)
                image_superset.contains(f(x))
                image_superset.contains(y)
            }
        }

        finite_constraint(set_image(s, f).contains)
        set_image(s, f).is_finite
    }
}

/// The image of a set with at most `n` elements has at most `n` elements.
theorem set_image_cardinality_at_most[T, U](s: Set[T], f: T -> U, n: Nat) {
    s.cardinality_at_most(n) implies set_image(s, f).cardinality_at_most(n)
} by {
    if s.cardinality_at_most(n) {
        let superset: List[T] satisfy {
            superset.contains_set(s) and superset.length <= n
        }

        let image_superset = superset.map(f)

        forall(y: U) {
            if set_image(s, f).contains(y) {
                set_image_contains_witness(s, f, y)
                let x: T satisfy {
                    s.contains(x) and y = f(x)
                }
                superset.contains_set(s) = forall(z: T) {
                    s.contains(z) implies superset.contains(z)
                }
                s.contains(x) implies superset.contains(x)
                superset.contains(x)
                map_contains_of_contains(superset, f, x)
                image_superset.contains(f(x))
                image_superset.contains(y)
            }
        }

        image_superset.contains_set(set_image(s, f))
        map_length(superset, f)
        image_superset.length = superset.length
        image_superset.length <= n
        set_image(s, f).cardinality_at_most(n)
    }
}

/// An injective map admits a finite list of preimage witnesses over any finite image list.
theorem injective_image_list_has_preimage_witnesses[T, U](s: Set[T], f: T -> U, image_items: List[U]) {
    is_injective_fn(f) and image_items.contains_set(set_image(s, f)) implies exists(witnesses: List[T]) {
        witnesses.contains_set(s)
    }
} by {
    define p(items: List[U]) -> Bool {
        forall(a: Set[T]) {
            items.contains_set(set_image(a, f)) implies exists(witnesses: List[T]) {
                witnesses.contains_set(a)
            }
        }
    }

    forall(a: Set[T]) {
        if List.nil[U].contains_set(set_image(a, f)) {
            forall(x: T) {
                if a.contains(x) {
                    maps_into_set_image(a, f, x)
                    List.nil[U].contains_set(set_image(a, f)) = forall(y: U) {
                        set_image(a, f).contains(y) implies List.nil[U].contains(y)
                    }
                    set_image(a, f).contains(f(x)) implies List.nil[U].contains(f(x))
                    List.nil[U].contains(f(x))
                    false
                }
            }
            List.nil[T].contains_set(a)
            exists(witnesses: List[T]) {
                witnesses.contains_set(a)
            }
        }
    }
    p(List.nil[U])

    forall(head: U, tail: List[U]) {
        if p(tail) {
            forall(a: Set[T]) {
                if List.cons(head, tail).contains_set(set_image(a, f)) {
                    if set_image(a, f).contains(head) {
                        set_image_contains_witness(a, f, head)
                        let head_preimage: T satisfy {
                            a.contains(head_preimage) and head = f(head_preimage)
                        }
                        let rest = a.remove(head_preimage)

                        forall(y: U) {
                            if set_image(rest, f).contains(y) {
                                set_image_contains_witness(rest, f, y)
                                let x: T satisfy {
                                    rest.contains(x) and y = f(x)
                                }
                                rest.contains(x)
                                functional_remove(a.contains, head_preimage, x)
                                a.contains(x)
                                List.cons(head, tail).contains_set(set_image(a, f)) = forall(z: U) {
                                    set_image(a, f).contains(z) implies List.cons(head, tail).contains(z)
                                }
                                maps_into_set_image(a, f, x)
                                set_image(a, f).contains(f(x))
                                set_image(a, f).contains(y)
                                List.cons(head, tail).contains(y)
                                if head = y {
                                    head = f(x)
                                    head = f(head_preimage)
                                    f(x) = f(head_preimage)
                                    injective_fn_eq(f, x, head_preimage)
                                    x = head_preimage
                                    false
                                }
                                tail.contains(y)
                            }
                        }

                        tail.contains_set(set_image(rest, f))
                        let tail_witnesses: List[T] satisfy {
                            tail_witnesses.contains_set(rest)
                        }
                        let witnesses = List.cons(head_preimage, tail_witnesses)

                        forall(x: T) {
                            if a.contains(x) {
                                if x = head_preimage {
                                    witnesses.contains(x)
                                } else {
                                    functional_remove(a.contains, head_preimage, x)
                                    rest.contains(x)
                                    tail_witnesses.contains_set(rest) = forall(z: T) {
                                        rest.contains(z) implies tail_witnesses.contains(z)
                                    }
                                    rest.contains(x) implies tail_witnesses.contains(x)
                                    tail_witnesses.contains(x)
                                    witnesses.contains(x)
                                }
                            }
                        }
                        witnesses.contains_set(a)
                        exists(result: List[T]) {
                            result.contains_set(a)
                        }
                    } else {
                        forall(y: U) {
                            if set_image(a, f).contains(y) {
                                List.cons(head, tail).contains_set(set_image(a, f)) = forall(z: U) {
                                    set_image(a, f).contains(z) implies List.cons(head, tail).contains(z)
                                }
                                List.cons(head, tail).contains(y)
                                if head = y {
                                    set_image(a, f).contains(head)
                                    false
                                }
                                tail.contains(y)
                            }
                        }
                        tail.contains_set(set_image(a, f))
                        let witnesses: List[T] satisfy {
                            witnesses.contains_set(a)
                        }
                        exists(result: List[T]) {
                            result.contains_set(a)
                        }
                    }
                }
            }
            p(List.cons(head, tail))
        }
    }

    if is_injective_fn(f) and image_items.contains_set(set_image(s, f)) {
        p(image_items)
        exists(witnesses: List[T]) {
            witnesses.contains_set(s)
        }
    }
}

/// A set whose injective image is contained in a finite list is finite.
theorem finite_of_injective_image_contained_in_list[T, U](s: Set[T], f: T -> U, image_items: List[U]) {
    is_injective_fn(f) and image_items.contains_set(set_image(s, f)) implies s.is_finite
} by {
    if is_injective_fn(f) and image_items.contains_set(set_image(s, f)) {
        injective_image_list_has_preimage_witnesses(s, f, image_items)
        let witnesses: List[T] satisfy {
            witnesses.contains_set(s)
        }
        witnesses.contains_set(s) = forall(x: T) {
            s.contains(x) implies witnesses.contains(x)
        }
        finite_constraint(s.contains)
        s.is_finite
    }
}

/// The image of a preimage of a listed set is contained in that list.
theorem set_image_preimage_list_set_contained[T, U](f: T -> U, image_items: List[U]) {
    image_items.contains_set(set_image(set_preimage(f, list_set(image_items)), f))
} by {
    let s = set_preimage(f, list_set(image_items))
    forall(y: U) {
        if set_image(s, f).contains(y) {
            set_image_contains_witness(s, f, y)
            let x: T satisfy {
                s.contains(x) and y = f(x)
            }
            set_preimage_contains_eq(f, list_set(image_items), x)
            preimage_contains(f, list_set(image_items), x)
            list_set(image_items).contains(f(x))
            list_set_contains_eq(image_items, f(x))
            image_items.contains(f(x))
            image_items.contains(y)
        }
    }
}

/// The preimage of a list-finite set under an injective function is finite.
theorem injective_preimage_list_set_is_finite[T, U](f: T -> U, image_items: List[U]) {
    is_injective_fn(f) implies set_preimage(f, list_set(image_items)).is_finite
} by {
    if is_injective_fn(f) {
        let s = set_preimage(f, list_set(image_items))
        set_image_preimage_list_set_contained(f, image_items)
        image_items.contains_set(set_image(s, f))
        finite_of_injective_image_contained_in_list(s, f, image_items)
        s.is_finite
    }
}

/// A set with cardinality bounded by a natural number is finite.
theorem cardinality_at_most_implies_is_finite[K](s: Set[K], n: Nat) {
    s.cardinality_at_most(n) implies s.is_finite
} by {
    if s.cardinality_at_most(n) {
        let superset: List[K] satisfy {
            superset.contains_set(s) and superset.length <= n
        }

        forall(x: K) {
            if s.contains(x) {
                superset.contains_set(s) = forall(y: K) {
                    s.contains(y) implies superset.contains(y)
                }
                s.contains(x) implies superset.contains(x)
                superset.contains(x)
            }
        }

        finite_constraint(s.contains)
        s.is_finite
    }
}

/// A finite set has some cardinality bound.
theorem finite_implies_exists_cardinality_at_most[K](s: Set[K]) {
    s.is_finite implies exists(n: Nat) {
        s.cardinality_at_most(n)
    }
} by {
    if s.is_finite {
        finite_constraint(s.contains)
        let superset: List[K] satisfy {
            forall(x: K) {
                s.contains(x) implies superset.contains(x)
            }
        }

        let n = superset.length
        superset.contains_set(s)
        superset.length <= n
        s.cardinality_at_most(n)
        exists(n0: Nat) {
            s.cardinality_at_most(n0)
        }
    }
}

theorem cardinality_always_exists[K](s: Set[K]) {
    s.is_finite implies exists(n: Nat) {
        s.cardinality_is(n)
    }
} by {

    let (superset: List[K]) satisfy {
        forall(x: K) {
            s.contains(x) implies superset.contains(x)
        }
    }

    let n = superset.filter(s.contains).unique.length
    superset.contains_set(s)
    superset.filter(s.contains).unique.length = n
    exists(containing_list: List[K]) {
        containing_list.contains_set(s) and containing_list.filter(s.contains).unique.length = n
    }
    s.cardinality_is(n)
    exists(n0: Nat) {
        s.cardinality_is(n0)
    }
}

theorem union_is_at_most_length[K](a: Set[K], b: Set[K], n1: Nat, n2: Nat) {
    a.cardinality_at_most(n1) and b.cardinality_at_most(n2) implies a.union(b).cardinality_at_most(n1 + n2)
} by {
    let (superset_a: List[K]) satisfy {
        superset_a.contains_set(a) and superset_a.length <= n1
    }

    let (superset_b: List[K]) satisfy {
        superset_b.contains_set(b) and superset_b.length <= n2
    }

    let superset_union = superset_a + superset_b

    forall(x: K) {
        if a.union(b).contains(x) {
            elem_in_union(a, b, x)
            if a.contains(x) {
                superset_a.contains(x)
                superset_union.contains(x)
            } else {
                b.contains(x)
                superset_b.contains(x)
                superset_union.contains(x)
            }
        }
    }

    superset_union.contains_set(a.union(b))

    superset_a.length <= n1
    superset_b.length <= n2
    superset_a.length + superset_b.length <= n1 + n2
    superset_a.length + superset_b.length = (superset_a + superset_b).length
    superset_union.length <= n1 + n2
    a.union(b).cardinality_at_most(n1 + n2)
}

/// A subset has cardinality at most that of a finite containing set.
theorem subset_cardinality_at_most[K](a: Set[K], b: Set[K], n: Nat) {
    a.subset(b) and b.cardinality_at_most(n) implies a.cardinality_at_most(n)
} by {
    if a.subset(b) and b.cardinality_at_most(n) {
        let superset_b: List[K] satisfy {
            superset_b.contains_set(b) and superset_b.length <= n
        }

        forall(x: K) {
            if a.contains(x) {
                b.contains(x)
                superset_b.contains_set(b) = forall(y: K) {
                    b.contains(y) implies superset_b.contains(y)
                }
                b.contains(x) implies superset_b.contains(x)
                superset_b.contains(x)
            }
        }

        superset_b.contains_set(a)
        a.cardinality_at_most(n)
    }
}

theorem disjoint_union_is_length[K](a: Set[K], b: Set[K], n1: Nat, n2: Nat) {
    a.cardinality_is(n1) and b.cardinality_is(n2) and a.is_disjoint(b)
    implies
    a.union(b).cardinality_is(n1 + n2)
} by {
    let list_a: List[K] satisfy {
        list_a.contains_set(a) and list_a.filter(a.contains).unique.length = n1
    }

    let list_b: List[K] satisfy {
        list_b.contains_set(b) and list_b.filter(b.contains).unique.length = n2
    }

    let f_list_a = list_a.filter(a.contains).unique
    let f_list_b = list_b.filter(b.contains).unique

    let list_union = f_list_a + f_list_b

    forall(x: K) {
        f_list_a.contains(x) implies list_union.contains(x)

        f_list_b.contains(x) implies list_union.contains(x)
        b.contains(x) implies list_union.contains(x)

        a.contains(x) implies list_union.contains(x)
        a.union(b).contains(x) implies list_union.contains(x)
    }

    forall (x: K) {
        if a.contains(x) {
            a.union(b).contains(x) = list_union.contains(x)
        } else {
            if b.contains(x) {
                a.union(b).contains(x) = list_union.contains(x)
            } else {
                not f_list_a.contains(x)
                not f_list_b.contains(x)
                a.union(b).contains(x) = list_union.contains(x)
            }
        }
    }

    a.union(b).contains = list_union.contains

    f_list_b.is_unique
    forall(x: K) {
        f_list_a.contains(x) implies a.contains(x)
        f_list_b.contains(x) implies b.contains(x)
        not (a.contains(x) and b.contains(x))
        not (f_list_a.contains(x) and f_list_b.contains(x))
    }
    f_list_a.is_unique
    f_list_a.is_unique and f_list_b.is_unique and forall(x: K) { not (f_list_a.contains(x) and f_list_b.contains(x)) }
    (f_list_a + f_list_b).is_unique
    list_union.is_unique
    list_union.unique = list_union
    list_union.filter(list_union.contains) = list_union
    list_union.length = n1 + n2
    list_union.contains_set(a.union(b))
    list_union.filter(a.union(b).contains).unique.length = n1 + n2
}

/// Insertion is union with the corresponding singleton.
theorem insert_eq_union_singleton[K](s: Set[K], item: K) {
    s.insert(item) = s.union(Set[K].singleton(item))
} by {
    let u = s.insert(item)
    let v = s.union(Set[K].singleton(item))
    forall(x: K) {
        insert_contains_eq(s, item, x)
        union_contains_eq(s, Set[K].singleton(item), x)
        singleton_contains_eq(item, x)
        if u.contains(x) {
            if x = item {
                v.contains(x)
            } else {
                s.contains(x)
                v.contains(x)
            }
        }
        if v.contains(x) {
            if s.contains(x) {
                u.contains(x)
            } else {
                x = item
                u.contains(x)
            }
        }
        u.contains(x) = v.contains(x)
    }
    set_ext(u, v)
}

/// Removal is difference by the corresponding singleton.
theorem remove_eq_difference_singleton[K](s: Set[K], item: K) {
    s.remove(item) = s.difference(Set[K].singleton(item))
} by {
    let u = s.remove(item)
    let v = s.difference(Set[K].singleton(item))
    forall(x: K) {
        remove_contains_eq(s, item, x)
        difference_contains_eq(s, Set[K].singleton(item), x)
        singleton_contains_eq(item, x)
        if u.contains(x) {
            s.contains(x)
            x != item
            not Set[K].singleton(item).contains(x)
            v.contains(x)
        }
        if v.contains(x) {
            s.contains(x)
            not Set[K].singleton(item).contains(x)
            x != item
            u.contains(x)
        }
        u.contains(x) = v.contains(x)
    }
    set_ext(u, v)
}

/// Inserting a new element increases cardinality by one.
theorem insert_cardinality_is_suc_of_not_contains[K](s: Set[K], item: K, n: Nat) {
    s.cardinality_is(n) and not s.contains(item) implies s.insert(item).cardinality_is(n + Nat.1)
} by {
    if s.cardinality_is(n) and not s.contains(item) {
        singleton_set_cardinality_is_one(item)
        forall(x: K) {
            if s.contains(x) {
                singleton_contains_eq(item, x)
                not Set[K].singleton(item).contains(x)
            }
            not (s.contains(x) and Set[K].singleton(item).contains(x))
        }
        s.is_disjoint(Set[K].singleton(item))
        disjoint_union_is_length(s, Set[K].singleton(item), n, Nat.1)
        s.union(Set[K].singleton(item)).cardinality_is(n + Nat.1)
        insert_eq_union_singleton(s, item)
        s.insert(item).cardinality_is(n + Nat.1)
    }
}

/// Removing an element and then adding it back has the expected exact cardinality.
theorem remove_insert_cardinality_is_suc_of_contains[K](s: Set[K], item: K, n: Nat) {
    s.remove(item).cardinality_is(n) and s.contains(item) implies s.cardinality_is(n + Nat.1)
} by {
    if s.remove(item).cardinality_is(n) and s.contains(item) {
        not s.remove(item).contains(item)
        insert_cardinality_is_suc_of_not_contains(s.remove(item), item, n)
        s.remove(item).insert(item).cardinality_is(n + Nat.1)
        remove_then_insert(s, item)
        s.remove(item).insert(item) = s
        s.cardinality_is(n + Nat.1)
    }
}

theorem cardinality_is_well_defined[K](s: Set[K], n1: Nat, n2: Nat) {
    s.cardinality_is(n1) and s.cardinality_is(n2) implies n1 = n2
} by {
    let (list_1: List[K]) satisfy {
        list_1.contains_set(s) and list_1.filter(s.contains).unique.length = n1
    }
    let (list_2: List[K]) satisfy {
        list_2.contains_set(s) and list_2.filter(s.contains).unique.length = n2
    }

    let f_list_1 = list_1.filter(s.contains).unique
    let f_list_2 = list_2.filter(s.contains).unique

    forall(x: K) {
        if s.contains(x) {
            f_list_2.contains(x)
            f_list_1.contains(x) = f_list_2.contains(x)
        } else {
            not f_list_2.contains(x)
            f_list_1.contains(x) = f_list_2.contains(x)
        }
    }

    forall(x: K) {
        f_list_1.contains(x) implies f_list_2.contains(x)
        f_list_2.contains(x) implies f_list_1.contains(x)
    }
    f_list_1.unique = f_list_1
    f_list_2.unique = f_list_2
    f_list_1.unique.length <= f_list_2.unique.length
    f_list_2.unique.length <= f_list_1.unique.length
}

theorem cardinality_is_smallest_cardinality[K](s: Set[K], n: Nat) {
    s.cardinality_is(n) implies s.cardinality_at_most(n)
} by {
    if s.cardinality_is(n) {
        let (containing_list: List[K]) satisfy {
            containing_list.contains_set(s) and containing_list.filter(s.contains).unique.length = n
        }

        let f_containing_list = containing_list.filter(s.contains).unique

        forall(x: K) {
            s.contains(x) implies f_containing_list.contains(x)
        }

        f_containing_list.contains_set(s)
        f_containing_list.length <= n
        s.cardinality_at_most(n)
    }
}

theorem cardinality_is_implies_is_finite[K](s: Set[K], n: Nat) {
    s.cardinality_is(n) implies s.is_finite
} by {
    if s.cardinality_is(n) {
        let (containing_list: List[K]) satisfy {
            containing_list.contains_set(s) and containing_list.filter(s.contains).unique.length = n
        }

        forall(x: K) {
            s.contains(x) implies containing_list.contains(x)
        }
        finite_constraint(s.contains)
        s.is_finite = finite_constraint(s.contains)
        s.is_finite
    }
}

/// Removing an absent element preserves exact cardinality.
theorem remove_cardinality_is_of_not_contains[K](s: Set[K], item: K, n: Nat) {
    s.cardinality_is(n) and not s.contains(item) implies s.remove(item).cardinality_is(n)
} by {
    if s.cardinality_is(n) and not s.contains(item) {
        forall(x: K) {
            remove_contains_eq(s, item, x)
            if s.remove(item).contains(x) {
                s.contains(x)
            }
            if s.contains(x) {
                x != item
                s.remove(item).contains(x)
            }
            s.remove(item).contains(x) = s.contains(x)
        }
        set_ext(s.remove(item), s)
        s.remove(item) = s
        s.remove(item).cardinality_is(n)
    }
}

/// Removing a present element decreases exact cardinality by one.
theorem remove_cardinality_is_pred_of_contains[K](s: Set[K], item: K, n: Nat) {
    s.cardinality_is(n + Nat.1) and s.contains(item) implies s.remove(item).cardinality_is(n)
} by {
    if s.cardinality_is(n + Nat.1) and s.contains(item) {
        cardinality_is_implies_is_finite(s, n + Nat.1)
        remove_is_finite_of_finite(s, item)
        cardinality_always_exists(s.remove(item))
        let m: Nat satisfy {
            s.remove(item).cardinality_is(m)
        }
        remove_insert_cardinality_is_suc_of_contains(s, item, m)
        s.cardinality_is(m + Nat.1)
        cardinality_is_well_defined(s, n + Nat.1, m + Nat.1)
        n + Nat.1 = m + Nat.1
        n = m
        s.remove(item).cardinality_is(n)
    }
}

/// Intersecting a set with one of its supersets preserves exact cardinality.
theorem intersection_cardinality_is_of_subset_left[K](s: Set[K], t: Set[K], n: Nat) {
    s.subset(t) and s.cardinality_is(n) implies s.intersection(t).cardinality_is(n)
} by {
    if s.subset(t) and s.cardinality_is(n) {
        intersection_with_superset_is_self(s, t)
        s.intersection(t) = s
        s.intersection(t).cardinality_is(n)
    }
}

/// Intersecting a set with one of its subsets has the subset's exact cardinality.
theorem intersection_cardinality_is_of_subset_right[K](s: Set[K], t: Set[K], n: Nat) {
    t.subset(s) and t.cardinality_is(n) implies s.intersection(t).cardinality_is(n)
} by {
    if t.subset(s) and t.cardinality_is(n) {
        intersection_with_superset_is_self(t, s)
        t.intersection(s) = t
        intersection_comm(s, t)
        s.intersection(t) = t.intersection(s)
        s.intersection(t).cardinality_is(n)
    }
}

/// Difference by a disjoint set preserves exact cardinality.
theorem difference_cardinality_is_of_disjoint[K](s: Set[K], t: Set[K], n: Nat) {
    s.cardinality_is(n) and s.is_disjoint(t) implies s.difference(t).cardinality_is(n)
} by {
    if s.cardinality_is(n) and s.is_disjoint(t) {
        forall(x: K) {
            difference_contains_eq(s, t, x)
            if s.difference(t).contains(x) {
                s.contains(x)
            }
            if s.contains(x) {
                not (s.contains(x) and t.contains(x))
                not t.contains(x)
                s.difference(t).contains(x)
            }
            s.difference(t).contains(x) = s.contains(x)
        }
        set_ext(s.difference(t), s)
        s.difference(t) = s
        s.difference(t).cardinality_is(n)
    }
}

/// Difference by a superset has cardinality zero.
theorem difference_cardinality_is_zero_of_subset[K](s: Set[K], t: Set[K]) {
    s.subset(t) implies s.difference(t).cardinality_is(Nat.0)
} by {
    if s.subset(t) {
        forall(x: K) {
            if s.difference(t).contains(x) {
                difference_contains_eq(s, t, x)
                s.contains(x)
                t.contains(x)
                false
            }
            not s.difference(t).contains(x)
        }
        set_ext(s.difference(t), Set[K].empty_set)
        s.difference(t) = Set[K].empty_set
        empty_set_cardinality_is_zero[K]
        s.difference(t).cardinality_is(Nat.0)
    }
}

theorem empty_set_is_finite[T] {
    Set[T].empty_set.is_finite
}

theorem union_cardinality_with_difference[K](s: Set[K], t: Set[K], n_s: Nat, n_diff: Nat) {
    s.cardinality_is(n_s) and t.difference(s).cardinality_is(n_diff) implies s.union(t).cardinality_is(n_s + n_diff)
} by {
    s.is_disjoint(t.difference(s))
    s.union(t.difference(s)).cardinality_is(n_s + n_diff)
}

/// The intersection of two sets is disjoint from the left difference.
theorem intersection_is_disjoint_difference[K](s: Set[K], t: Set[K]) {
    s.intersection(t).is_disjoint(s.difference(t))
} by {
    let sit = s.intersection(t)
    let s_diff = s.difference(t)
    forall(x: K) {
        if sit.contains(x) {
            elem_in_intersection(s, t, x)
            t.contains(x)
            not s_diff.contains(x)
        }
        if s_diff.contains(x) {
            difference_contains_eq(s, t, x)
            not t.contains(x)
            not sit.contains(x)
        }
        not (sit.contains(x) and s_diff.contains(x))
    }
}

/// The union of the intersection with the left difference is the left set.
theorem intersection_union_difference_is_self[K](s: Set[K], t: Set[K]) {
    s.intersection(t).union(s.difference(t)) = s
} by {
    let sit = s.intersection(t)
    let s_diff = s.difference(t)

    forall(x: K) {
        if sit.union(s_diff).contains(x) {
            if sit.contains(x) {
                elem_in_intersection(s, t, x)
                s.contains(x)
            } else {
                s_diff.contains(x)
                difference_contains_eq(s, t, x)
                s.contains(x)
            }
        }
    }
    sit.union(s_diff).subset(s)

    forall(x: K) {
        if s.contains(x) {
            if t.contains(x) {
                elem_in_intersection(s, t, x)
                sit.contains(x)
                elem_in_union(sit, s_diff, x)
            } else {
                s_diff.contains(x)
                elem_in_union(sit, s_diff, x)
            }
        }
    }
    s.subset(sit.union(s_diff))
    double_inclusion(sit.union(s_diff), s)
}

/// Exact cardinalities for the intersection and left difference add to the left cardinality.
theorem intersection_difference_cardinality_adds[K](s: Set[K], t: Set[K], n_inter: Nat, n_diff: Nat) {
    s.intersection(t).cardinality_is(n_inter) and s.difference(t).cardinality_is(n_diff)
    implies s.cardinality_is(n_inter + n_diff)
} by {
    if s.intersection(t).cardinality_is(n_inter) and s.difference(t).cardinality_is(n_diff) {
        let sit = s.intersection(t)
        let s_diff = s.difference(t)
        intersection_is_disjoint_difference(s, t)
        sit.is_disjoint(s_diff)
        disjoint_union_is_length(sit, s_diff, n_inter, n_diff)
        sit.union(s_diff).cardinality_is(n_inter + n_diff)
        intersection_union_difference_is_self(s, t)
        sit.union(s_diff) = s
        s.cardinality_is(n_inter + n_diff)
    }
}

/// The left difference has cardinality equal to the left cardinality minus the intersection cardinality.
theorem difference_cardinality_is_sub_intersection[K](s: Set[K], t: Set[K], n_s: Nat, n_inter: Nat) {
    s.cardinality_is(n_s) and s.intersection(t).cardinality_is(n_inter)
    implies s.difference(t).cardinality_is(n_s - n_inter)
} by {
    if s.cardinality_is(n_s) and s.intersection(t).cardinality_is(n_inter) {
        cardinality_is_implies_is_finite(s, n_s)
        difference_is_finite_of_finite(s, t)
        cardinality_always_exists(s.difference(t))
        let n_diff: Nat satisfy {
            s.difference(t).cardinality_is(n_diff)
        }
        intersection_difference_cardinality_adds(s, t, n_inter, n_diff)
        s.cardinality_is(n_inter + n_diff)
        cardinality_is_well_defined(s, n_s, n_inter + n_diff)
        n_inter + n_diff = n_s
        add_imp_sub(n_inter, n_diff, n_s)
        n_s - n_inter = n_diff
        s.difference(t).cardinality_is(n_s - n_inter)
    }
}

/// The intersection has cardinality equal to the left cardinality minus the left-difference cardinality.
theorem intersection_cardinality_is_sub_difference[K](s: Set[K], t: Set[K], n_s: Nat, n_diff: Nat) {
    s.cardinality_is(n_s) and s.difference(t).cardinality_is(n_diff)
    implies s.intersection(t).cardinality_is(n_s - n_diff)
} by {
    if s.cardinality_is(n_s) and s.difference(t).cardinality_is(n_diff) {
        cardinality_is_implies_is_finite(s, n_s)
        intersection_is_finite_of_finite(s, s)
        subset_is_finite_of_finite(s.intersection(t), s)
        cardinality_always_exists(s.intersection(t))
        let n_inter: Nat satisfy {
            s.intersection(t).cardinality_is(n_inter)
        }
        intersection_difference_cardinality_adds(s, t, n_inter, n_diff)
        s.cardinality_is(n_inter + n_diff)
        cardinality_is_well_defined(s, n_s, n_inter + n_diff)
        n_inter + n_diff = n_s
        add_imp_sub(n_inter, n_diff, n_s)
        n_s - n_diff = n_inter
        s.intersection(t).cardinality_is(n_s - n_diff)
    }
}

theorem inclusion_exclusion[K](s: Set[K], t: Set[K], n_s: Nat, n_t: Nat, n_inter: Nat) {
    s.cardinality_is(n_s) and t.cardinality_is(n_t) and s.intersection(t).cardinality_is(n_inter)
    implies s.union(t).cardinality_is(n_s + n_t - n_inter)
} by {
    let sut = s.union(t)
    let sit = s.intersection(t)
    let sym_diff = sut.difference(sit)

    // (S \ (S \cap T)) \cup (S \cap T) = S
    let s_diff = s.difference(sit)
    sit.union(s.difference(sit)) = sit.union(s)
    sit.subset(s)
    sit.union(s) = s
    sit.union(s_diff) = s
    s_diff.is_finite implies exists(n: Nat) {
        s_diff.cardinality_is(n)
    }
    let n_s_diff: Nat satisfy {
        s_diff.cardinality_is(n_s_diff)
    }
    sit.union(s_diff).cardinality_is(n_inter + n_s_diff)

    // T is disjoint from (S \ (S \cap T))
    forall (x: K) {
        s_diff.contains(x) implies not sit.contains(x)
        if t.contains(x) {
            if s_diff.contains(x) {
                s.contains(x)
                sit.contains(x)
            }
        }
        not (t.contains(x) and s_diff.contains(x))
    }
    t.is_disjoint(s_diff)

    t.union(s_diff).cardinality_is(n_t + n_s_diff)
    t.union(s_diff) = t.union(s)
    t.union(s) = s.union(t)
    n_t + n_s_diff = n_t + n_s - n_inter
}

theorem set_has_exact_containing_list[K](s: Set[K]) {
    s.is_finite implies exists(containing_list: List[K]) {
        forall(x: K) {
            s.contains(x) = containing_list.contains(x)
        } and containing_list.is_unique
    }
} by {
    if s.is_finite {
        let (superset: List[K]) satisfy {
            forall(x: K) {
                s.contains(x) implies superset.contains(x)
            }
        }

        let filtered = superset.filter(s.contains).unique

        forall(x: K) {
            filtered.contains(x) implies s.contains(x)
            s.contains(x) = filtered.contains(x)
        }
        filtered.is_unique

        exists(containing_list: List[K]) {
            forall(x: K) {
                s.contains(x) = filtered.contains(x)
            } and filtered.is_unique
        }
    }
}

/// A finite set is equal to the set associated to some list.
theorem finite_set_has_list_set[K](s: Set[K]) {
    s.is_finite implies exists(items: List[K]) {
        list_set(items) = s
    }
} by {
    if s.is_finite {
        set_has_exact_containing_list(s)
        let items: List[K] satisfy {
            forall(x: K) {
                s.contains(x) = items.contains(x)
            } and items.is_unique
        }
        forall(x: K) {
            list_set_contains_eq(items, x)
            list_set(items).contains(x) = items.contains(x)
            s.contains(x) = list_set(items).contains(x)
        }
        set_ext(s, list_set(items))
        s = list_set(items)
        exists(result: List[K]) {
            list_set(result) = s
        }
    }
}

/// A finite set is equal to the set associated to some unique list.
theorem finite_set_has_unique_list_set[K](s: Set[K]) {
    s.is_finite implies exists(items: List[K]) {
        list_set(items) = s and items.is_unique
    }
} by {
    if s.is_finite {
        set_has_exact_containing_list(s)
        let items: List[K] satisfy {
            forall(x: K) {
                s.contains(x) = items.contains(x)
            } and items.is_unique
        }
        forall(x: K) {
            list_set_contains_eq(items, x)
            list_set(items).contains(x) = items.contains(x)
            s.contains(x) = list_set(items).contains(x)
        }
        set_ext(s, list_set(items))
        s = list_set(items)
        exists(result: List[K]) {
            list_set(result) = s and result.is_unique
        }
    }
}

/// A set with exact finite cardinality is represented by a unique list of that length.
theorem set_cardinality_has_exact_unique_list[K](s: Set[K], n: Nat) {
    s.cardinality_is(n) implies exists(items: List[K]) {
        list_set(items) = s and items.is_unique and items.length = n
    }
} by {
    if s.cardinality_is(n) {
        cardinality_is_implies_is_finite(s, n)
        set_has_exact_containing_list(s)
        let items: List[K] satisfy {
            forall(x: K) {
                s.contains(x) = items.contains(x)
            } and items.is_unique
        }

        forall(x: K) {
            list_set_contains_eq(items, x)
            list_set(items).contains(x) = items.contains(x)
            s.contains(x) = list_set(items).contains(x)
        }
        set_ext(list_set(items), s)
        list_set(items) = s
        unique_list_set_cardinality_is_length(items)
        list_set(items).cardinality_is(items.length)
        s.cardinality_is(items.length)
        cardinality_is_well_defined(s, n, items.length)
        items.length = n
        exists(result: List[K]) {
            list_set(result) = s and result.is_unique and result.length = n
        }
    }
}

/// An injective image has the same exact cardinality as the source set.
theorem set_image_cardinality_is_of_injective[T, U](s: Set[T], f: T -> U, n: Nat) {
    is_injective_fn(f) and s.cardinality_is(n) implies set_image(s, f).cardinality_is(n)
} by {
    if is_injective_fn(f) and s.cardinality_is(n) {
        set_cardinality_has_exact_unique_list(s, n)
        let items: List[T] satisfy {
            list_set(items) = s and items.is_unique and items.length = n
        }

        let image_items = items.map(f)

        forall(y: U) {
            if set_image(s, f).contains(y) {
                set_image_contains_witness(s, f, y)
                let x: T satisfy {
                    s.contains(x) and y = f(x)
                }
                list_set_contains_eq(items, x)
                list_set(items).contains(x)
                items.contains(x)
                map_contains_of_contains(items, f, x)
                image_items.contains(f(x))
                image_items.contains(y)
            }
            if image_items.contains(y) {
                map_contains(items, f, y)
                let x: T satisfy {
                    items.contains(x) and f(x) = y
                }
                list_set_contains_eq(items, x)
                list_set(items).contains(x)
                s.contains(x)
                maps_into_set_image(s, f, x)
                set_image(s, f).contains(f(x))
                set_image(s, f).contains(y)
            }
            list_set_contains_eq(image_items, y)
            list_set(image_items).contains(y) = image_items.contains(y)
            set_image(s, f).contains(y) = image_items.contains(y)
            set_image(s, f).contains(y) = list_set(image_items).contains(y)
        }
        set_ext(set_image(s, f), list_set(image_items))
        set_image(s, f) = list_set(image_items)

        injective_map_is_unique(items, f)
        image_items.is_unique
        unique_list_set_cardinality_is_length(image_items)
        list_set(image_items).cardinality_is(image_items.length)
        set_image(s, f).cardinality_is(image_items.length)

        map_length(items, f)
        image_items.length = items.length
        image_items.length = n
        set_image(s, f).cardinality_is(n)
    }
}
