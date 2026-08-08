/// Pointwise algebraic operations on finite products of function spaces.

from pair import Pair, pair_ext
from algebra.add import Add
from algebra.add_semigroup import AddSemigroup
from algebra.add_comm_semigroup import AddCommSemigroup
from algebra.add_monoid import AddMonoid
from algebra.add_group import AddGroup
from algebra.mul import Mul
from algebra.semigroup import Semigroup
from algebra.comm_semigroup import CommSemigroup
from algebra.one import One
from algebra.zero import Zero
from algebra.neg import Neg
from algebra.monoid.monoid import Monoid
from algebra.group import Group
from semiring import Semiring
from data.basic.function_algebra import pointwise_add, pointwise_mul, pointwise_zero, pointwise_one,
    pointwise_neg, pointwise_inverse, pointwise_add_assoc, pointwise_add_comm,
    pointwise_add_zero_right, pointwise_add_zero_left, pointwise_add_neg_right,
    pointwise_add_neg_left, pointwise_mul_assoc, pointwise_mul_comm, pointwise_mul_one_right,
    pointwise_mul_one_left, pointwise_mul_inverse_right, pointwise_mul_inverse_left,
    pointwise_mul_distrib_left, pointwise_mul_distrib_right, pointwise_mul_zero_right,
    pointwise_mul_zero_left
from algebra.product_algebra import pair_add, pair_mul, pair_zero, pair_one, pair_neg, pair_inverse

/// The value of a binary product of functions at one point.
define function_pair_apply[T, A, B](p: Pair[T -> A, T -> B], t: T) -> Pair[A, B] {
    Pair.new(p.first(t), p.second(t))
}

/// The binary product of two functions.
define function_pair[T, A, B](f: T -> A, g: T -> B) -> Pair[T -> A, T -> B] {
    Pair.new(f, g)
}

/// The pointwise sum on a binary product of function spaces.
define function_pair_add[T, A: Add, B: Add](
    p: Pair[T -> A, T -> B],
    q: Pair[T -> A, T -> B]
) -> Pair[T -> A, T -> B] {
    Pair.new(pointwise_add(p.first, q.first), pointwise_add(p.second, q.second))
}

/// The pointwise product on a binary product of function spaces.
define function_pair_mul[T, A: Mul, B: Mul](
    p: Pair[T -> A, T -> B],
    q: Pair[T -> A, T -> B]
) -> Pair[T -> A, T -> B] {
    Pair.new(pointwise_mul(p.first, q.first), pointwise_mul(p.second, q.second))
}

/// The pointwise zero on a binary product of function spaces.
let function_pair_zero[T, A: Zero, B: Zero]: Pair[T -> A, T -> B] =
    Pair.new(pointwise_zero[T, A], pointwise_zero[T, B])

/// The pointwise one on a binary product of function spaces.
let function_pair_one[T, A: One, B: One]: Pair[T -> A, T -> B] =
    Pair.new(pointwise_one[T, A], pointwise_one[T, B])

/// The pointwise negation on a binary product of function spaces.
define function_pair_neg[T, A: Neg, B: Neg](
    p: Pair[T -> A, T -> B]
) -> Pair[T -> A, T -> B] {
    Pair.new(pointwise_neg(p.first), pointwise_neg(p.second))
}

/// The pointwise inverse on a binary product of function spaces.
define function_pair_inverse[T, A: Group, B: Group](
    p: Pair[T -> A, T -> B]
) -> Pair[T -> A, T -> B] {
    Pair.new(pointwise_inverse(p.first), pointwise_inverse(p.second))
}

/// Evaluation of a binary product of functions has the expected first coordinate.
theorem function_pair_apply_first[T, A, B](p: Pair[T -> A, T -> B], t: T) {
    function_pair_apply(p, t).first = p.first(t)
}

/// Evaluation of a binary product of functions has the expected second coordinate.
theorem function_pair_apply_second[T, A, B](p: Pair[T -> A, T -> B], t: T) {
    function_pair_apply(p, t).second = p.second(t)
}

/// The binary product of two functions has the expected first coordinate.
theorem function_pair_first[T, A, B](f: T -> A, g: T -> B) {
    function_pair(f, g).first = f
}

/// The binary product of two functions has the expected second coordinate.
theorem function_pair_second[T, A, B](f: T -> A, g: T -> B) {
    function_pair(f, g).second = g
}

/// Evaluation of a binary product of two functions gives the pair of values.
theorem function_pair_apply_pair[T, A, B](f: T -> A, g: T -> B, t: T) {
    function_pair_apply(function_pair(f, g), t) = Pair.new(f(t), g(t))
}

/// Function pairs are determined by their two coordinates.
theorem function_pair_eta[T, A, B](p: Pair[T -> A, T -> B]) {
    function_pair(p.first, p.second) = p
} by {
    let q = function_pair(p.first, p.second)
    pair_ext(q, p)
}

/// The first coordinate of the pointwise sum is the pointwise sum of first coordinates.
theorem function_pair_add_first[T, A: Add, B: Add](
    p: Pair[T -> A, T -> B],
    q: Pair[T -> A, T -> B]
) {
    function_pair_add(p, q).first = pointwise_add(p.first, q.first)
}

/// The second coordinate of the pointwise sum is the pointwise sum of second coordinates.
theorem function_pair_add_second[T, A: Add, B: Add](
    p: Pair[T -> A, T -> B],
    q: Pair[T -> A, T -> B]
) {
    function_pair_add(p, q).second = pointwise_add(p.second, q.second)
}

/// The first coordinate of the pointwise product is the pointwise product of first coordinates.
theorem function_pair_mul_first[T, A: Mul, B: Mul](
    p: Pair[T -> A, T -> B],
    q: Pair[T -> A, T -> B]
) {
    function_pair_mul(p, q).first = pointwise_mul(p.first, q.first)
}

/// The second coordinate of the pointwise product is the pointwise product of second coordinates.
theorem function_pair_mul_second[T, A: Mul, B: Mul](
    p: Pair[T -> A, T -> B],
    q: Pair[T -> A, T -> B]
) {
    function_pair_mul(p, q).second = pointwise_mul(p.second, q.second)
}

/// The first coordinate of the pointwise zero is the pointwise zero.
theorem function_pair_zero_first[T, A: Zero, B: Zero] {
    function_pair_zero[T, A, B].first = pointwise_zero[T, A]
}

/// The second coordinate of the pointwise zero is the pointwise zero.
theorem function_pair_zero_second[T, A: Zero, B: Zero] {
    function_pair_zero[T, A, B].second = pointwise_zero[T, B]
}

/// The first coordinate of the pointwise one is the pointwise one.
theorem function_pair_one_first[T, A: One, B: One] {
    function_pair_one[T, A, B].first = pointwise_one[T, A]
}

/// The second coordinate of the pointwise one is the pointwise one.
theorem function_pair_one_second[T, A: One, B: One] {
    function_pair_one[T, A, B].second = pointwise_one[T, B]
}

/// The first coordinate of the pointwise negation is the pointwise negation.
theorem function_pair_neg_first[T, A: Neg, B: Neg](p: Pair[T -> A, T -> B]) {
    function_pair_neg(p).first = pointwise_neg(p.first)
}

/// The second coordinate of the pointwise negation is the pointwise negation.
theorem function_pair_neg_second[T, A: Neg, B: Neg](p: Pair[T -> A, T -> B]) {
    function_pair_neg(p).second = pointwise_neg(p.second)
}

/// The first coordinate of the pointwise inverse is the pointwise inverse.
theorem function_pair_inverse_first[T, A: Group, B: Group](p: Pair[T -> A, T -> B]) {
    function_pair_inverse(p).first = pointwise_inverse(p.first)
}

/// The second coordinate of the pointwise inverse is the pointwise inverse.
theorem function_pair_inverse_second[T, A: Group, B: Group](p: Pair[T -> A, T -> B]) {
    function_pair_inverse(p).second = pointwise_inverse(p.second)
}

/// Evaluation transports pointwise sums to componentwise sums of values.
theorem function_pair_add_apply[T, A: Add, B: Add](
    p: Pair[T -> A, T -> B],
    q: Pair[T -> A, T -> B],
    t: T
) {
    function_pair_apply(function_pair_add(p, q), t) =
    pair_add(function_pair_apply(p, t), function_pair_apply(q, t))
} by {
    let lhs = function_pair_apply(function_pair_add(p, q), t)
    let rhs = pair_add(function_pair_apply(p, t), function_pair_apply(q, t))
    function_pair_add(p, q).first = pointwise_add(p.first, q.first)
    lhs.first = pointwise_add(p.first, q.first, t)
    lhs.first = p.first(t) + q.first(t)
    rhs.first = function_pair_apply(p, t).first + function_pair_apply(q, t).first
    rhs.first = p.first(t) + q.first(t)
    lhs.first = rhs.first
    lhs.second = function_pair_add(p, q).second(t)
    function_pair_add(p, q).second = pointwise_add(p.second, q.second)
    lhs.second = pointwise_add(p.second, q.second, t)
    lhs.second = p.second(t) + q.second(t)
    rhs.second = function_pair_apply(p, t).second + function_pair_apply(q, t).second
    rhs.second = p.second(t) + q.second(t)
    lhs.second = rhs.second
    pair_ext(lhs, rhs)
}

/// Evaluation transports pointwise products to componentwise products of values.
theorem function_pair_mul_apply[T, A: Mul, B: Mul](
    p: Pair[T -> A, T -> B],
    q: Pair[T -> A, T -> B],
    t: T
) {
    function_pair_apply(function_pair_mul(p, q), t) =
    pair_mul(function_pair_apply(p, t), function_pair_apply(q, t))
} by {
    let lhs = function_pair_apply(function_pair_mul(p, q), t)
    let rhs = pair_mul(function_pair_apply(p, t), function_pair_apply(q, t))
    lhs.first = function_pair_mul(p, q).first(t)
    function_pair_mul(p, q).first = pointwise_mul(p.first, q.first)
    lhs.first = pointwise_mul(p.first, q.first, t)
    lhs.first = p.first(t) * q.first(t)
    rhs.first = function_pair_apply(p, t).first * function_pair_apply(q, t).first
    rhs.first = p.first(t) * q.first(t)
    lhs.first = rhs.first
    lhs.second = function_pair_mul(p, q).second(t)
    function_pair_mul(p, q).second = pointwise_mul(p.second, q.second)
    lhs.second = pointwise_mul(p.second, q.second, t)
    lhs.second = p.second(t) * q.second(t)
    rhs.second = function_pair_apply(p, t).second * function_pair_apply(q, t).second
    rhs.second = p.second(t) * q.second(t)
    lhs.second = rhs.second
    pair_ext(lhs, rhs)
}

/// Evaluation transports the pointwise zero to the componentwise zero of values.
theorem function_pair_zero_apply[T, A: Zero, B: Zero](t: T) {
    function_pair_apply(function_pair_zero[T, A, B], t) = pair_zero[A, B]
} by {
    let lhs = function_pair_apply(function_pair_zero[T, A, B], t)
    lhs.first = function_pair_zero[T, A, B].first(t)
    function_pair_zero[T, A, B].first = pointwise_zero[T, A]
    lhs.first = pointwise_zero[T, A](t)
    lhs.first = A.0
    lhs.second = function_pair_zero[T, A, B].second(t)
    function_pair_zero[T, A, B].second = pointwise_zero[T, B]
    lhs.second = pointwise_zero[T, B](t)
    lhs.second = B.0
    pair_ext(lhs, pair_zero[A, B])
}

/// Evaluation transports the pointwise one to the componentwise one of values.
theorem function_pair_one_apply[T, A: One, B: One](t: T) {
    function_pair_apply(function_pair_one[T, A, B], t) = pair_one[A, B]
} by {
    let lhs = function_pair_apply(function_pair_one[T, A, B], t)
    lhs.first = function_pair_one[T, A, B].first(t)
    function_pair_one[T, A, B].first = pointwise_one[T, A]
    lhs.first = pointwise_one[T, A](t)
    lhs.first = A.1
    lhs.second = function_pair_one[T, A, B].second(t)
    function_pair_one[T, A, B].second = pointwise_one[T, B]
    lhs.second = pointwise_one[T, B](t)
    lhs.second = B.1
    pair_ext(lhs, pair_one[A, B])
}

/// Evaluation transports pointwise negation to componentwise negation of values.
theorem function_pair_neg_apply[T, A: Neg, B: Neg](p: Pair[T -> A, T -> B], t: T) {
    function_pair_apply(function_pair_neg(p), t) = pair_neg(function_pair_apply(p, t))
} by {
    let lhs = function_pair_apply(function_pair_neg(p), t)
    let rhs = pair_neg(function_pair_apply(p, t))
    lhs.first = function_pair_neg(p).first(t)
    function_pair_neg(p).first = pointwise_neg(p.first)
    lhs.first = pointwise_neg(p.first, t)
    lhs.first = -p.first(t)
    rhs.first = -function_pair_apply(p, t).first
    rhs.first = -p.first(t)
    lhs.first = rhs.first
    lhs.second = function_pair_neg(p).second(t)
    function_pair_neg(p).second = pointwise_neg(p.second)
    lhs.second = pointwise_neg(p.second, t)
    lhs.second = -p.second(t)
    rhs.second = -function_pair_apply(p, t).second
    rhs.second = -p.second(t)
    lhs.second = rhs.second
    pair_ext(lhs, rhs)
}

/// Evaluation transports pointwise inversion to componentwise inversion of values.
theorem function_pair_inverse_apply[T, A: Group, B: Group](p: Pair[T -> A, T -> B], t: T) {
    function_pair_apply(function_pair_inverse(p), t) = pair_inverse(function_pair_apply(p, t))
} by {
    let lhs = function_pair_apply(function_pair_inverse(p), t)
    let rhs = pair_inverse(function_pair_apply(p, t))
    lhs.first = function_pair_inverse(p).first(t)
    function_pair_inverse(p).first = pointwise_inverse(p.first)
    lhs.first = pointwise_inverse(p.first, t)
    lhs.first = p.first(t).inverse
    rhs.first = function_pair_apply(p, t).first.inverse
    rhs.first = p.first(t).inverse
    lhs.first = rhs.first
    lhs.second = function_pair_inverse(p).second(t)
    function_pair_inverse(p).second = pointwise_inverse(p.second)
    lhs.second = pointwise_inverse(p.second, t)
    lhs.second = p.second(t).inverse
    rhs.second = function_pair_apply(p, t).second.inverse
    rhs.second = p.second(t).inverse
    lhs.second = rhs.second
    pair_ext(lhs, rhs)
}

/// Pointwise addition is associative on a binary product of function spaces.
theorem function_pair_add_assoc[T, A: AddSemigroup, B: AddSemigroup](
    p: Pair[T -> A, T -> B],
    q: Pair[T -> A, T -> B],
    r: Pair[T -> A, T -> B]
) {
    function_pair_add(p, function_pair_add(q, r)) =
    function_pair_add(function_pair_add(p, q), r)
} by {
    let lhs = function_pair_add(p, function_pair_add(q, r))
    let rhs = function_pair_add(function_pair_add(p, q), r)
    lhs.first = pointwise_add(p.first, function_pair_add(q, r).first)
    function_pair_add(q, r).first = pointwise_add(q.first, r.first)
    lhs.first = pointwise_add(p.first, pointwise_add(q.first, r.first))
    rhs.first = pointwise_add(function_pair_add(p, q).first, r.first)
    function_pair_add(p, q).first = pointwise_add(p.first, q.first)
    rhs.first = pointwise_add(pointwise_add(p.first, q.first), r.first)
    pointwise_add_assoc(p.first, q.first, r.first)
    lhs.first = rhs.first
    lhs.second = pointwise_add(p.second, function_pair_add(q, r).second)
    function_pair_add(q, r).second = pointwise_add(q.second, r.second)
    lhs.second = pointwise_add(p.second, pointwise_add(q.second, r.second))
    rhs.second = pointwise_add(function_pair_add(p, q).second, r.second)
    function_pair_add(p, q).second = pointwise_add(p.second, q.second)
    rhs.second = pointwise_add(pointwise_add(p.second, q.second), r.second)
    pointwise_add_assoc(p.second, q.second, r.second)
    lhs.second = rhs.second
    pair_ext(lhs, rhs)
}

/// Pointwise addition is commutative on a binary product of function spaces.
theorem function_pair_add_comm[T, A: AddCommSemigroup, B: AddCommSemigroup](
    p: Pair[T -> A, T -> B],
    q: Pair[T -> A, T -> B]
) {
    function_pair_add(p, q) = function_pair_add(q, p)
} by {
    let lhs = function_pair_add(p, q)
    let rhs = function_pair_add(q, p)
    lhs.first = pointwise_add(p.first, q.first)
    rhs.first = pointwise_add(q.first, p.first)
    pointwise_add_comm(p.first, q.first)
    lhs.first = rhs.first
    lhs.second = pointwise_add(p.second, q.second)
    rhs.second = pointwise_add(q.second, p.second)
    pointwise_add_comm(p.second, q.second)
    lhs.second = rhs.second
    pair_ext(lhs, rhs)
}

/// Adding the pointwise zero on the right changes no coordinate.
theorem function_pair_add_zero_right[T, A: AddMonoid, B: AddMonoid](
    p: Pair[T -> A, T -> B]
) {
    function_pair_add(p, function_pair_zero[T, A, B]) = p
} by {
    let lhs = function_pair_add(p, function_pair_zero[T, A, B])
    lhs.first = pointwise_add(p.first, function_pair_zero[T, A, B].first)
    function_pair_zero[T, A, B].first = pointwise_zero[T, A]
    lhs.first = pointwise_add(p.first, pointwise_zero[T, A])
    pointwise_add_zero_right(p.first)
    lhs.first = p.first
    lhs.second = pointwise_add(p.second, function_pair_zero[T, A, B].second)
    function_pair_zero[T, A, B].second = pointwise_zero[T, B]
    lhs.second = pointwise_add(p.second, pointwise_zero[T, B])
    pointwise_add_zero_right(p.second)
    lhs.second = p.second
    pair_ext(lhs, p)
}

/// Adding the pointwise zero on the left changes no coordinate.
theorem function_pair_add_zero_left[T, A: AddMonoid, B: AddMonoid](
    p: Pair[T -> A, T -> B]
) {
    function_pair_add(function_pair_zero[T, A, B], p) = p
} by {
    let lhs = function_pair_add(function_pair_zero[T, A, B], p)
    lhs.first = pointwise_add(function_pair_zero[T, A, B].first, p.first)
    function_pair_zero[T, A, B].first = pointwise_zero[T, A]
    lhs.first = pointwise_add(pointwise_zero[T, A], p.first)
    pointwise_add_zero_left(p.first)
    lhs.first = p.first
    lhs.second = pointwise_add(function_pair_zero[T, A, B].second, p.second)
    function_pair_zero[T, A, B].second = pointwise_zero[T, B]
    lhs.second = pointwise_add(pointwise_zero[T, B], p.second)
    pointwise_add_zero_left(p.second)
    lhs.second = p.second
    pair_ext(lhs, p)
}

/// Pointwise negation is a right additive inverse on a binary product of function spaces.
theorem function_pair_add_neg_right[T, A: AddGroup, B: AddGroup](
    p: Pair[T -> A, T -> B]
) {
    function_pair_add(p, function_pair_neg(p)) = function_pair_zero[T, A, B]
} by {
    let lhs = function_pair_add(p, function_pair_neg(p))
    lhs.first = pointwise_add(p.first, function_pair_neg(p).first)
    function_pair_neg(p).first = pointwise_neg(p.first)
    lhs.first = pointwise_add(p.first, pointwise_neg(p.first))
    pointwise_add_neg_right(p.first)
    lhs.first = pointwise_zero[T, A]
    function_pair_zero[T, A, B].first = pointwise_zero[T, A]
    lhs.second = pointwise_add(p.second, function_pair_neg(p).second)
    function_pair_neg(p).second = pointwise_neg(p.second)
    lhs.second = pointwise_add(p.second, pointwise_neg(p.second))
    pointwise_add_neg_right(p.second)
    lhs.second = pointwise_zero[T, B]
    function_pair_zero[T, A, B].second = pointwise_zero[T, B]
    pair_ext(lhs, function_pair_zero[T, A, B])
}

/// Pointwise negation is a left additive inverse on a binary product of function spaces.
theorem function_pair_add_neg_left[T, A: AddGroup, B: AddGroup](
    p: Pair[T -> A, T -> B]
) {
    function_pair_add(function_pair_neg(p), p) = function_pair_zero[T, A, B]
} by {
    let lhs = function_pair_add(function_pair_neg(p), p)
    lhs.first = pointwise_add(function_pair_neg(p).first, p.first)
    function_pair_neg(p).first = pointwise_neg(p.first)
    lhs.first = pointwise_add(pointwise_neg(p.first), p.first)
    pointwise_add_neg_left(p.first)
    lhs.first = pointwise_zero[T, A]
    function_pair_zero[T, A, B].first = pointwise_zero[T, A]
    lhs.second = pointwise_add(function_pair_neg(p).second, p.second)
    function_pair_neg(p).second = pointwise_neg(p.second)
    lhs.second = pointwise_add(pointwise_neg(p.second), p.second)
    pointwise_add_neg_left(p.second)
    lhs.second = pointwise_zero[T, B]
    function_pair_zero[T, A, B].second = pointwise_zero[T, B]
    pair_ext(lhs, function_pair_zero[T, A, B])
}

/// Pointwise multiplication is associative on a binary product of function spaces.
theorem function_pair_mul_assoc[T, A: Semigroup, B: Semigroup](
    p: Pair[T -> A, T -> B],
    q: Pair[T -> A, T -> B],
    r: Pair[T -> A, T -> B]
) {
    function_pair_mul(p, function_pair_mul(q, r)) =
    function_pair_mul(function_pair_mul(p, q), r)
} by {
    let lhs = function_pair_mul(p, function_pair_mul(q, r))
    let rhs = function_pair_mul(function_pair_mul(p, q), r)
    lhs.first = pointwise_mul(p.first, function_pair_mul(q, r).first)
    function_pair_mul(q, r).first = pointwise_mul(q.first, r.first)
    lhs.first = pointwise_mul(p.first, pointwise_mul(q.first, r.first))
    rhs.first = pointwise_mul(function_pair_mul(p, q).first, r.first)
    function_pair_mul(p, q).first = pointwise_mul(p.first, q.first)
    rhs.first = pointwise_mul(pointwise_mul(p.first, q.first), r.first)
    pointwise_mul_assoc(p.first, q.first, r.first)
    lhs.first = rhs.first
    lhs.second = pointwise_mul(p.second, function_pair_mul(q, r).second)
    function_pair_mul(q, r).second = pointwise_mul(q.second, r.second)
    lhs.second = pointwise_mul(p.second, pointwise_mul(q.second, r.second))
    rhs.second = pointwise_mul(function_pair_mul(p, q).second, r.second)
    function_pair_mul(p, q).second = pointwise_mul(p.second, q.second)
    rhs.second = pointwise_mul(pointwise_mul(p.second, q.second), r.second)
    pointwise_mul_assoc(p.second, q.second, r.second)
    lhs.second = rhs.second
    pair_ext(lhs, rhs)
}

/// Pointwise multiplication is commutative on a binary product of function spaces.
theorem function_pair_mul_comm[T, A: CommSemigroup, B: CommSemigroup](
    p: Pair[T -> A, T -> B],
    q: Pair[T -> A, T -> B]
) {
    function_pair_mul(p, q) = function_pair_mul(q, p)
} by {
    let lhs = function_pair_mul(p, q)
    let rhs = function_pair_mul(q, p)
    lhs.first = pointwise_mul(p.first, q.first)
    rhs.first = pointwise_mul(q.first, p.first)
    pointwise_mul_comm(p.first, q.first)
    lhs.first = rhs.first
    lhs.second = pointwise_mul(p.second, q.second)
    rhs.second = pointwise_mul(q.second, p.second)
    pointwise_mul_comm(p.second, q.second)
    lhs.second = rhs.second
    pair_ext(lhs, rhs)
}

/// Multiplying by the pointwise one on the right changes no coordinate.
theorem function_pair_mul_one_right[T, A: Monoid, B: Monoid](
    p: Pair[T -> A, T -> B]
) {
    function_pair_mul(p, function_pair_one[T, A, B]) = p
} by {
    let lhs = function_pair_mul(p, function_pair_one[T, A, B])
    lhs.first = pointwise_mul(p.first, function_pair_one[T, A, B].first)
    function_pair_one[T, A, B].first = pointwise_one[T, A]
    lhs.first = pointwise_mul(p.first, pointwise_one[T, A])
    pointwise_mul_one_right(p.first)
    lhs.first = p.first
    lhs.second = pointwise_mul(p.second, function_pair_one[T, A, B].second)
    function_pair_one[T, A, B].second = pointwise_one[T, B]
    lhs.second = pointwise_mul(p.second, pointwise_one[T, B])
    pointwise_mul_one_right(p.second)
    lhs.second = p.second
    pair_ext(lhs, p)
}

/// Multiplying by the pointwise one on the left changes no coordinate.
theorem function_pair_mul_one_left[T, A: Monoid, B: Monoid](
    p: Pair[T -> A, T -> B]
) {
    function_pair_mul(function_pair_one[T, A, B], p) = p
} by {
    let lhs = function_pair_mul(function_pair_one[T, A, B], p)
    lhs.first = pointwise_mul(function_pair_one[T, A, B].first, p.first)
    function_pair_one[T, A, B].first = pointwise_one[T, A]
    lhs.first = pointwise_mul(pointwise_one[T, A], p.first)
    pointwise_mul_one_left(p.first)
    lhs.first = p.first
    lhs.second = pointwise_mul(function_pair_one[T, A, B].second, p.second)
    function_pair_one[T, A, B].second = pointwise_one[T, B]
    lhs.second = pointwise_mul(pointwise_one[T, B], p.second)
    pointwise_mul_one_left(p.second)
    lhs.second = p.second
    pair_ext(lhs, p)
}

/// Pointwise inversion is a right inverse on a binary product of function spaces.
theorem function_pair_mul_inverse_right[T, A: Group, B: Group](
    p: Pair[T -> A, T -> B]
) {
    function_pair_mul(p, function_pair_inverse(p)) = function_pair_one[T, A, B]
} by {
    let lhs = function_pair_mul(p, function_pair_inverse(p))
    lhs.first = pointwise_mul(p.first, function_pair_inverse(p).first)
    function_pair_inverse(p).first = pointwise_inverse(p.first)
    lhs.first = pointwise_mul(p.first, pointwise_inverse(p.first))
    pointwise_mul_inverse_right(p.first)
    lhs.first = pointwise_one[T, A]
    function_pair_one[T, A, B].first = pointwise_one[T, A]
    lhs.second = pointwise_mul(p.second, function_pair_inverse(p).second)
    function_pair_inverse(p).second = pointwise_inverse(p.second)
    lhs.second = pointwise_mul(p.second, pointwise_inverse(p.second))
    pointwise_mul_inverse_right(p.second)
    lhs.second = pointwise_one[T, B]
    function_pair_one[T, A, B].second = pointwise_one[T, B]
    pair_ext(lhs, function_pair_one[T, A, B])
}

/// Pointwise inversion is a left inverse on a binary product of function spaces.
theorem function_pair_mul_inverse_left[T, A: Group, B: Group](
    p: Pair[T -> A, T -> B]
) {
    function_pair_mul(function_pair_inverse(p), p) = function_pair_one[T, A, B]
} by {
    let lhs = function_pair_mul(function_pair_inverse(p), p)
    lhs.first = pointwise_mul(function_pair_inverse(p).first, p.first)
    function_pair_inverse(p).first = pointwise_inverse(p.first)
    lhs.first = pointwise_mul(pointwise_inverse(p.first), p.first)
    pointwise_mul_inverse_left(p.first)
    lhs.first = pointwise_one[T, A]
    function_pair_one[T, A, B].first = pointwise_one[T, A]
    lhs.second = pointwise_mul(function_pair_inverse(p).second, p.second)
    function_pair_inverse(p).second = pointwise_inverse(p.second)
    lhs.second = pointwise_mul(pointwise_inverse(p.second), p.second)
    pointwise_mul_inverse_left(p.second)
    lhs.second = pointwise_one[T, B]
    function_pair_one[T, A, B].second = pointwise_one[T, B]
    pair_ext(lhs, function_pair_one[T, A, B])
}

/// Pointwise multiplication distributes over pointwise addition on the left.
theorem function_pair_mul_distrib_left[T, A: Semiring, B: Semiring](
    p: Pair[T -> A, T -> B],
    q: Pair[T -> A, T -> B],
    r: Pair[T -> A, T -> B]
) {
    function_pair_mul(p, function_pair_add(q, r)) =
    function_pair_add(function_pair_mul(p, q), function_pair_mul(p, r))
} by {
    let lhs = function_pair_mul(p, function_pair_add(q, r))
    let rhs = function_pair_add(function_pair_mul(p, q), function_pair_mul(p, r))
    lhs.first = pointwise_mul(p.first, function_pair_add(q, r).first)
    function_pair_add(q, r).first = pointwise_add(q.first, r.first)
    lhs.first = pointwise_mul(p.first, pointwise_add(q.first, r.first))
    rhs.first = pointwise_add(function_pair_mul(p, q).first, function_pair_mul(p, r).first)
    function_pair_mul(p, q).first = pointwise_mul(p.first, q.first)
    function_pair_mul(p, r).first = pointwise_mul(p.first, r.first)
    rhs.first = pointwise_add(pointwise_mul(p.first, q.first), pointwise_mul(p.first, r.first))
    pointwise_mul_distrib_left(p.first, q.first, r.first)
    lhs.first = rhs.first
    lhs.second = pointwise_mul(p.second, function_pair_add(q, r).second)
    function_pair_add(q, r).second = pointwise_add(q.second, r.second)
    lhs.second = pointwise_mul(p.second, pointwise_add(q.second, r.second))
    rhs.second = pointwise_add(function_pair_mul(p, q).second, function_pair_mul(p, r).second)
    function_pair_mul(p, q).second = pointwise_mul(p.second, q.second)
    function_pair_mul(p, r).second = pointwise_mul(p.second, r.second)
    rhs.second = pointwise_add(pointwise_mul(p.second, q.second), pointwise_mul(p.second, r.second))
    pointwise_mul_distrib_left(p.second, q.second, r.second)
    lhs.second = rhs.second
    pair_ext(lhs, rhs)
}

/// Pointwise multiplication distributes over pointwise addition on the right.
theorem function_pair_mul_distrib_right[T, A: Semiring, B: Semiring](
    p: Pair[T -> A, T -> B],
    q: Pair[T -> A, T -> B],
    r: Pair[T -> A, T -> B]
) {
    function_pair_mul(function_pair_add(p, q), r) =
    function_pair_add(function_pair_mul(p, r), function_pair_mul(q, r))
} by {
    let lhs = function_pair_mul(function_pair_add(p, q), r)
    let rhs = function_pair_add(function_pair_mul(p, r), function_pair_mul(q, r))
    lhs.first = pointwise_mul(function_pair_add(p, q).first, r.first)
    function_pair_add(p, q).first = pointwise_add(p.first, q.first)
    lhs.first = pointwise_mul(pointwise_add(p.first, q.first), r.first)
    rhs.first = pointwise_add(function_pair_mul(p, r).first, function_pair_mul(q, r).first)
    function_pair_mul(p, r).first = pointwise_mul(p.first, r.first)
    function_pair_mul(q, r).first = pointwise_mul(q.first, r.first)
    rhs.first = pointwise_add(pointwise_mul(p.first, r.first), pointwise_mul(q.first, r.first))
    pointwise_mul_distrib_right(p.first, q.first, r.first)
    lhs.first = rhs.first
    lhs.second = pointwise_mul(function_pair_add(p, q).second, r.second)
    function_pair_add(p, q).second = pointwise_add(p.second, q.second)
    lhs.second = pointwise_mul(pointwise_add(p.second, q.second), r.second)
    rhs.second = pointwise_add(function_pair_mul(p, r).second, function_pair_mul(q, r).second)
    function_pair_mul(p, r).second = pointwise_mul(p.second, r.second)
    function_pair_mul(q, r).second = pointwise_mul(q.second, r.second)
    rhs.second = pointwise_add(pointwise_mul(p.second, r.second), pointwise_mul(q.second, r.second))
    pointwise_mul_distrib_right(p.second, q.second, r.second)
    lhs.second = rhs.second
    pair_ext(lhs, rhs)
}

/// Multiplying by the pointwise zero on the right gives the pointwise zero.
theorem function_pair_mul_zero_right[T, A: Semiring, B: Semiring](
    p: Pair[T -> A, T -> B]
) {
    function_pair_mul(p, function_pair_zero[T, A, B]) = function_pair_zero[T, A, B]
} by {
    let lhs = function_pair_mul(p, function_pair_zero[T, A, B])
    lhs.first = pointwise_mul(p.first, function_pair_zero[T, A, B].first)
    function_pair_zero[T, A, B].first = pointwise_zero[T, A]
    lhs.first = pointwise_mul(p.first, pointwise_zero[T, A])
    pointwise_mul_zero_right(p.first)
    lhs.first = pointwise_zero[T, A]
    lhs.second = pointwise_mul(p.second, function_pair_zero[T, A, B].second)
    function_pair_zero[T, A, B].second = pointwise_zero[T, B]
    lhs.second = pointwise_mul(p.second, pointwise_zero[T, B])
    pointwise_mul_zero_right(p.second)
    lhs.second = pointwise_zero[T, B]
    pair_ext(lhs, function_pair_zero[T, A, B])
}

/// Multiplying by the pointwise zero on the left gives the pointwise zero.
theorem function_pair_mul_zero_left[T, A: Semiring, B: Semiring](
    p: Pair[T -> A, T -> B]
) {
    function_pair_mul(function_pair_zero[T, A, B], p) = function_pair_zero[T, A, B]
} by {
    let lhs = function_pair_mul(function_pair_zero[T, A, B], p)
    lhs.first = pointwise_mul(function_pair_zero[T, A, B].first, p.first)
    function_pair_zero[T, A, B].first = pointwise_zero[T, A]
    lhs.first = pointwise_mul(pointwise_zero[T, A], p.first)
    pointwise_mul_zero_left(p.first)
    lhs.first = pointwise_zero[T, A]
    lhs.second = pointwise_mul(function_pair_zero[T, A, B].second, p.second)
    function_pair_zero[T, A, B].second = pointwise_zero[T, B]
    lhs.second = pointwise_mul(pointwise_zero[T, B], p.second)
    pointwise_mul_zero_left(p.second)
    lhs.second = pointwise_zero[T, B]
    pair_ext(lhs, function_pair_zero[T, A, B])
}

/// The value of a ternary product of functions at one point.
define function_triple_apply[T, A, B, C](
    p: Pair[T -> A, Pair[T -> B, T -> C]],
    t: T
) -> Pair[A, Pair[B, C]] {
    Pair.new(p.first(t), Pair.new(p.second.first(t), p.second.second(t)))
}

/// The ternary product of three functions.
define function_triple[T, A, B, C](
    f: T -> A,
    g: T -> B,
    h: T -> C
) -> Pair[T -> A, Pair[T -> B, T -> C]] {
    Pair.new(f, Pair.new(g, h))
}

/// The pointwise sum on a ternary product of function spaces.
define function_triple_add[T, A: Add, B: Add, C: Add](
    p: Pair[T -> A, Pair[T -> B, T -> C]],
    q: Pair[T -> A, Pair[T -> B, T -> C]]
) -> Pair[T -> A, Pair[T -> B, T -> C]] {
    Pair.new(pointwise_add(p.first, q.first),
        Pair.new(pointwise_add(p.second.first, q.second.first),
            pointwise_add(p.second.second, q.second.second)))
}

/// The pointwise product on a ternary product of function spaces.
define function_triple_mul[T, A: Mul, B: Mul, C: Mul](
    p: Pair[T -> A, Pair[T -> B, T -> C]],
    q: Pair[T -> A, Pair[T -> B, T -> C]]
) -> Pair[T -> A, Pair[T -> B, T -> C]] {
    Pair.new(pointwise_mul(p.first, q.first),
        Pair.new(pointwise_mul(p.second.first, q.second.first),
            pointwise_mul(p.second.second, q.second.second)))
}

/// The pointwise zero on a ternary product of function spaces.
let function_triple_zero[T, A: Zero, B: Zero, C: Zero]:
    Pair[T -> A, Pair[T -> B, T -> C]] =
    Pair.new(pointwise_zero[T, A], Pair.new(pointwise_zero[T, B], pointwise_zero[T, C]))

/// The pointwise one on a ternary product of function spaces.
let function_triple_one[T, A: One, B: One, C: One]:
    Pair[T -> A, Pair[T -> B, T -> C]] =
    Pair.new(pointwise_one[T, A], Pair.new(pointwise_one[T, B], pointwise_one[T, C]))

/// The pointwise negation on a ternary product of function spaces.
define function_triple_neg[T, A: Neg, B: Neg, C: Neg](
    p: Pair[T -> A, Pair[T -> B, T -> C]]
) -> Pair[T -> A, Pair[T -> B, T -> C]] {
    Pair.new(pointwise_neg(p.first),
        Pair.new(pointwise_neg(p.second.first), pointwise_neg(p.second.second)))
}

/// The pointwise inverse on a ternary product of function spaces.
define function_triple_inverse[T, A: Group, B: Group, C: Group](
    p: Pair[T -> A, Pair[T -> B, T -> C]]
) -> Pair[T -> A, Pair[T -> B, T -> C]] {
    Pair.new(pointwise_inverse(p.first),
        Pair.new(pointwise_inverse(p.second.first), pointwise_inverse(p.second.second)))
}

/// Evaluation of a ternary product of functions has the expected first coordinate.
theorem function_triple_apply_first[T, A, B, C](
    p: Pair[T -> A, Pair[T -> B, T -> C]],
    t: T
) {
    function_triple_apply(p, t).first = p.first(t)
}

/// Evaluation of a ternary product of functions has the expected second coordinate.
theorem function_triple_apply_second_first[T, A, B, C](
    p: Pair[T -> A, Pair[T -> B, T -> C]],
    t: T
) {
    function_triple_apply(p, t).second.first = p.second.first(t)
}

/// Evaluation of a ternary product of functions has the expected third coordinate.
theorem function_triple_apply_second_second[T, A, B, C](
    p: Pair[T -> A, Pair[T -> B, T -> C]],
    t: T
) {
    function_triple_apply(p, t).second.second = p.second.second(t)
}

/// The ternary product of three functions has the expected first coordinate.
theorem function_triple_first[T, A, B, C](f: T -> A, g: T -> B, h: T -> C) {
    function_triple(f, g, h).first = f
}

/// The ternary product of three functions has the expected second coordinate.
theorem function_triple_second_first[T, A, B, C](f: T -> A, g: T -> B, h: T -> C) {
    function_triple(f, g, h).second.first = g
}

/// The ternary product of three functions has the expected third coordinate.
theorem function_triple_second_second[T, A, B, C](f: T -> A, g: T -> B, h: T -> C) {
    function_triple(f, g, h).second.second = h
}

/// Function triples are determined by their three coordinates.
theorem function_triple_eta[T, A, B, C](p: Pair[T -> A, Pair[T -> B, T -> C]]) {
    function_triple(p.first, p.second.first, p.second.second) = p
} by {
    let q = function_triple(p.first, p.second.first, p.second.second)
    q.first = p.first
    q.second.first = p.second.first
    q.second.second = p.second.second
    q.second = p.second
    pair_ext(q.second, p.second)
    pair_ext(q, p)
}

/// The first coordinate of a ternary pointwise sum is the pointwise sum of first coordinates.
theorem function_triple_add_first[T, A: Add, B: Add, C: Add](
    p: Pair[T -> A, Pair[T -> B, T -> C]],
    q: Pair[T -> A, Pair[T -> B, T -> C]]
) {
    function_triple_add(p, q).first = pointwise_add(p.first, q.first)
}

/// The first coordinate of a ternary pointwise product is the pointwise product of first coordinates.
theorem function_triple_mul_first[T, A: Mul, B: Mul, C: Mul](
    p: Pair[T -> A, Pair[T -> B, T -> C]],
    q: Pair[T -> A, Pair[T -> B, T -> C]]
) {
    function_triple_mul(p, q).first = pointwise_mul(p.first, q.first)
}

/// Ternary function products are determined by their three coordinates.
theorem function_triple_ext[T, A, B, C](
    p: Pair[T -> A, Pair[T -> B, T -> C]],
    q: Pair[T -> A, Pair[T -> B, T -> C]]
) {
    p.first = q.first and p.second.first = q.second.first and
    p.second.second = q.second.second implies p = q
} by {
    if p.first = q.first and p.second.first = q.second.first and
    p.second.second = q.second.second {
        p.second = q.second
        pair_ext(p.second, q.second)
        pair_ext(p, q)
    }
}

/// The second coordinate of a ternary pointwise sum is the pointwise sum of second coordinates.
theorem function_triple_add_second_first[T, A: Add, B: Add, C: Add](
    p: Pair[T -> A, Pair[T -> B, T -> C]],
    q: Pair[T -> A, Pair[T -> B, T -> C]]
) {
    function_triple_add(p, q).second.first = pointwise_add(p.second.first, q.second.first)
} by {
    function_triple_add(p, q).second =
    Pair.new(pointwise_add(p.second.first, q.second.first),
        pointwise_add(p.second.second, q.second.second))
}

/// The third coordinate of a ternary pointwise sum is the pointwise sum of third coordinates.
theorem function_triple_add_second_second[T, A: Add, B: Add, C: Add](
    p: Pair[T -> A, Pair[T -> B, T -> C]],
    q: Pair[T -> A, Pair[T -> B, T -> C]]
) {
    function_triple_add(p, q).second.second = pointwise_add(p.second.second, q.second.second)
} by {
    function_triple_add(p, q).second =
    Pair.new(pointwise_add(p.second.first, q.second.first),
        pointwise_add(p.second.second, q.second.second))
}

/// The second coordinate of a ternary pointwise product is the pointwise product of second coordinates.
theorem function_triple_mul_second_first[T, A: Mul, B: Mul, C: Mul](
    p: Pair[T -> A, Pair[T -> B, T -> C]],
    q: Pair[T -> A, Pair[T -> B, T -> C]]
) {
    function_triple_mul(p, q).second.first = pointwise_mul(p.second.first, q.second.first)
} by {
    function_triple_mul(p, q).second =
    Pair.new(pointwise_mul(p.second.first, q.second.first),
        pointwise_mul(p.second.second, q.second.second))
}

/// The third coordinate of a ternary pointwise product is the pointwise product of third coordinates.
theorem function_triple_mul_second_second[T, A: Mul, B: Mul, C: Mul](
    p: Pair[T -> A, Pair[T -> B, T -> C]],
    q: Pair[T -> A, Pair[T -> B, T -> C]]
) {
    function_triple_mul(p, q).second.second = pointwise_mul(p.second.second, q.second.second)
} by {
    function_triple_mul(p, q).second =
    Pair.new(pointwise_mul(p.second.first, q.second.first),
        pointwise_mul(p.second.second, q.second.second))
}

/// The first coordinate of the ternary pointwise zero is the pointwise zero.
theorem function_triple_zero_first[T, A: Zero, B: Zero, C: Zero] {
    function_triple_zero[T, A, B, C].first = pointwise_zero[T, A]
}

/// The second coordinate of the ternary pointwise zero is the pointwise zero.
theorem function_triple_zero_second_first[T, A: Zero, B: Zero, C: Zero] {
    function_triple_zero[T, A, B, C].second.first = pointwise_zero[T, B]
}

/// The third coordinate of the ternary pointwise zero is the pointwise zero.
theorem function_triple_zero_second_second[T, A: Zero, B: Zero, C: Zero] {
    function_triple_zero[T, A, B, C].second.second = pointwise_zero[T, C]
}

/// The first coordinate of the ternary pointwise one is the pointwise one.
theorem function_triple_one_first[T, A: One, B: One, C: One] {
    function_triple_one[T, A, B, C].first = pointwise_one[T, A]
}

/// The second coordinate of the ternary pointwise one is the pointwise one.
theorem function_triple_one_second_first[T, A: One, B: One, C: One] {
    function_triple_one[T, A, B, C].second.first = pointwise_one[T, B]
}

/// The third coordinate of the ternary pointwise one is the pointwise one.
theorem function_triple_one_second_second[T, A: One, B: One, C: One] {
    function_triple_one[T, A, B, C].second.second = pointwise_one[T, C]
}

/// The first coordinate of ternary pointwise negation is pointwise negation.
theorem function_triple_neg_first[T, A: Neg, B: Neg, C: Neg](
    p: Pair[T -> A, Pair[T -> B, T -> C]]
) {
    function_triple_neg(p).first = pointwise_neg(p.first)
}

/// The second coordinate of ternary pointwise negation is pointwise negation.
theorem function_triple_neg_second_first[T, A: Neg, B: Neg, C: Neg](
    p: Pair[T -> A, Pair[T -> B, T -> C]]
) {
    function_triple_neg(p).second.first = pointwise_neg(p.second.first)
}

/// The third coordinate of ternary pointwise negation is pointwise negation.
theorem function_triple_neg_second_second[T, A: Neg, B: Neg, C: Neg](
    p: Pair[T -> A, Pair[T -> B, T -> C]]
) {
    function_triple_neg(p).second.second = pointwise_neg(p.second.second)
}

/// The first coordinate of ternary pointwise inversion is pointwise inversion.
theorem function_triple_inverse_first[T, A: Group, B: Group, C: Group](
    p: Pair[T -> A, Pair[T -> B, T -> C]]
) {
    function_triple_inverse(p).first = pointwise_inverse(p.first)
}

/// The second coordinate of ternary pointwise inversion is pointwise inversion.
theorem function_triple_inverse_second_first[T, A: Group, B: Group, C: Group](
    p: Pair[T -> A, Pair[T -> B, T -> C]]
) {
    function_triple_inverse(p).second.first = pointwise_inverse(p.second.first)
}

/// The third coordinate of ternary pointwise inversion is pointwise inversion.
theorem function_triple_inverse_second_second[T, A: Group, B: Group, C: Group](
    p: Pair[T -> A, Pair[T -> B, T -> C]]
) {
    function_triple_inverse(p).second.second = pointwise_inverse(p.second.second)
}

/// Evaluation transports ternary pointwise sums to sums of values.
theorem function_triple_add_apply[T, A: Add, B: Add, C: Add](
    p: Pair[T -> A, Pair[T -> B, T -> C]],
    q: Pair[T -> A, Pair[T -> B, T -> C]],
    t: T
) {
    function_triple_apply(function_triple_add(p, q), t) =
    Pair.new(p.first(t) + q.first(t),
        Pair.new(p.second.first(t) + q.second.first(t),
            p.second.second(t) + q.second.second(t)))
} by {
    let lhs = function_triple_apply(function_triple_add(p, q), t)
    let rhs = Pair.new(p.first(t) + q.first(t),
        Pair.new(p.second.first(t) + q.second.first(t),
            p.second.second(t) + q.second.second(t)))
    lhs.first = function_triple_add(p, q).first(t)
    function_triple_add(p, q).first = pointwise_add(p.first, q.first)
    lhs.first = pointwise_add(p.first, q.first, t)
    lhs.first = p.first(t) + q.first(t)
    lhs.first = rhs.first
    lhs.second.first = function_triple_add(p, q).second.first(t)
    function_triple_add(p, q).second.first = pointwise_add(p.second.first, q.second.first)
    lhs.second.first = pointwise_add(p.second.first, q.second.first, t)
    lhs.second.first = p.second.first(t) + q.second.first(t)
    rhs.second = Pair.new(p.second.first(t) + q.second.first(t),
        p.second.second(t) + q.second.second(t))
    rhs.second.first = p.second.first(t) + q.second.first(t)
    lhs.second.first = rhs.second.first
    lhs.second.second = function_triple_add(p, q).second.second(t)
    function_triple_add(p, q).second.second = pointwise_add(p.second.second, q.second.second)
    lhs.second.second = pointwise_add(p.second.second, q.second.second, t)
    lhs.second.second = p.second.second(t) + q.second.second(t)
    rhs.second.second = p.second.second(t) + q.second.second(t)
    lhs.second.second = rhs.second.second
    lhs.second = rhs.second
    pair_ext(lhs.second, rhs.second)
    pair_ext(lhs, rhs)
}

/// Evaluation transports ternary pointwise products to products of values.
theorem function_triple_mul_apply[T, A: Mul, B: Mul, C: Mul](
    p: Pair[T -> A, Pair[T -> B, T -> C]],
    q: Pair[T -> A, Pair[T -> B, T -> C]],
    t: T
) {
    function_triple_apply(function_triple_mul(p, q), t) =
    Pair.new(p.first(t) * q.first(t),
        Pair.new(p.second.first(t) * q.second.first(t),
            p.second.second(t) * q.second.second(t)))
} by {
    let lhs = function_triple_apply(function_triple_mul(p, q), t)
    let rhs = Pair.new(p.first(t) * q.first(t),
        Pair.new(p.second.first(t) * q.second.first(t),
            p.second.second(t) * q.second.second(t)))
    lhs.first = function_triple_mul(p, q).first(t)
    function_triple_mul(p, q).first = pointwise_mul(p.first, q.first)
    lhs.first = pointwise_mul(p.first, q.first, t)
    lhs.first = p.first(t) * q.first(t)
    lhs.first = rhs.first
    lhs.second.first = function_triple_mul(p, q).second.first(t)
    function_triple_mul(p, q).second.first = pointwise_mul(p.second.first, q.second.first)
    lhs.second.first = pointwise_mul(p.second.first, q.second.first, t)
    lhs.second.first = p.second.first(t) * q.second.first(t)
    rhs.second = Pair.new(p.second.first(t) * q.second.first(t),
        p.second.second(t) * q.second.second(t))
    rhs.second.first = p.second.first(t) * q.second.first(t)
    lhs.second.first = rhs.second.first
    lhs.second.second = function_triple_mul(p, q).second.second(t)
    function_triple_mul(p, q).second.second = pointwise_mul(p.second.second, q.second.second)
    lhs.second.second = pointwise_mul(p.second.second, q.second.second, t)
    lhs.second.second = p.second.second(t) * q.second.second(t)
    rhs.second.second = p.second.second(t) * q.second.second(t)
    lhs.second.second = rhs.second.second
    lhs.second = rhs.second
    pair_ext(lhs.second, rhs.second)
    pair_ext(lhs, rhs)
}

/// Evaluation transports the ternary pointwise zero to zeros of values.
theorem function_triple_zero_apply[T, A: Zero, B: Zero, C: Zero](t: T) {
    function_triple_apply(function_triple_zero[T, A, B, C], t) =
    Pair.new(A.0, Pair.new(B.0, C.0))
} by {
    let lhs = function_triple_apply(function_triple_zero[T, A, B, C], t)
    let rhs = Pair.new(A.0, Pair.new(B.0, C.0))
    lhs.first = function_triple_zero[T, A, B, C].first(t)
    function_triple_zero[T, A, B, C].first = pointwise_zero[T, A]
    lhs.first = pointwise_zero[T, A](t)
    lhs.first = A.0
    lhs.first = rhs.first
    lhs.second.first = function_triple_zero[T, A, B, C].second.first(t)
    function_triple_zero[T, A, B, C].second.first = pointwise_zero[T, B]
    lhs.second.first = pointwise_zero[T, B](t)
    lhs.second.first = B.0
    rhs.second = Pair.new(B.0, C.0)
    lhs.second.first = rhs.second.first
    lhs.second.second = function_triple_zero[T, A, B, C].second.second(t)
    function_triple_zero[T, A, B, C].second.second = pointwise_zero[T, C]
    lhs.second.second = pointwise_zero[T, C](t)
    lhs.second.second = C.0
    lhs.second.second = rhs.second.second
    lhs.second = rhs.second
    pair_ext(lhs.second, rhs.second)
    pair_ext(lhs, rhs)
}

/// Evaluation transports the ternary pointwise one to ones of values.
theorem function_triple_one_apply[T, A: One, B: One, C: One](t: T) {
    function_triple_apply(function_triple_one[T, A, B, C], t) =
    Pair.new(A.1, Pair.new(B.1, C.1))
} by {
    let lhs = function_triple_apply(function_triple_one[T, A, B, C], t)
    let rhs = Pair.new(A.1, Pair.new(B.1, C.1))
    lhs.first = function_triple_one[T, A, B, C].first(t)
    function_triple_one[T, A, B, C].first = pointwise_one[T, A]
    lhs.first = pointwise_one[T, A](t)
    lhs.first = A.1
    lhs.first = rhs.first
    lhs.second.first = function_triple_one[T, A, B, C].second.first(t)
    function_triple_one[T, A, B, C].second.first = pointwise_one[T, B]
    lhs.second.first = pointwise_one[T, B](t)
    lhs.second.first = B.1
    rhs.second = Pair.new(B.1, C.1)
    lhs.second.first = rhs.second.first
    lhs.second.second = function_triple_one[T, A, B, C].second.second(t)
    function_triple_one[T, A, B, C].second.second = pointwise_one[T, C]
    lhs.second.second = pointwise_one[T, C](t)
    lhs.second.second = C.1
    lhs.second.second = rhs.second.second
    lhs.second = rhs.second
    pair_ext(lhs.second, rhs.second)
    pair_ext(lhs, rhs)
}

/// Evaluation transports ternary pointwise negation to negation of values.
theorem function_triple_neg_apply[T, A: Neg, B: Neg, C: Neg](
    p: Pair[T -> A, Pair[T -> B, T -> C]],
    t: T
) {
    function_triple_apply(function_triple_neg(p), t) =
    Pair.new(-p.first(t), Pair.new(-p.second.first(t), -p.second.second(t)))
} by {
    let lhs = function_triple_apply(function_triple_neg(p), t)
    let rhs = Pair.new(-p.first(t), Pair.new(-p.second.first(t), -p.second.second(t)))
    lhs.first = function_triple_neg(p).first(t)
    function_triple_neg(p).first = pointwise_neg(p.first)
    lhs.first = pointwise_neg(p.first, t)
    lhs.first = -p.first(t)
    lhs.first = rhs.first
    lhs.second.first = function_triple_neg(p).second.first(t)
    function_triple_neg(p).second.first = pointwise_neg(p.second.first)
    lhs.second.first = pointwise_neg(p.second.first, t)
    lhs.second.first = -p.second.first(t)
    rhs.second = Pair.new(-p.second.first(t), -p.second.second(t))
    lhs.second.first = rhs.second.first
    lhs.second.second = function_triple_neg(p).second.second(t)
    function_triple_neg(p).second.second = pointwise_neg(p.second.second)
    lhs.second.second = pointwise_neg(p.second.second, t)
    lhs.second.second = -p.second.second(t)
    lhs.second.second = rhs.second.second
    lhs.second = rhs.second
    pair_ext(lhs.second, rhs.second)
    pair_ext(lhs, rhs)
}

/// Evaluation transports ternary pointwise inversion to inversion of values.
theorem function_triple_inverse_apply[T, A: Group, B: Group, C: Group](
    p: Pair[T -> A, Pair[T -> B, T -> C]],
    t: T
) {
    function_triple_apply(function_triple_inverse(p), t) =
    Pair.new(p.first(t).inverse,
        Pair.new(p.second.first(t).inverse, p.second.second(t).inverse))
} by {
    let lhs = function_triple_apply(function_triple_inverse(p), t)
    let rhs = Pair.new(p.first(t).inverse,
        Pair.new(p.second.first(t).inverse, p.second.second(t).inverse))
    lhs.first = function_triple_inverse(p).first(t)
    function_triple_inverse(p).first = pointwise_inverse(p.first)
    lhs.first = pointwise_inverse(p.first, t)
    lhs.first = p.first(t).inverse
    lhs.first = rhs.first
    lhs.second.first = function_triple_inverse(p).second.first(t)
    function_triple_inverse(p).second.first = pointwise_inverse(p.second.first)
    lhs.second.first = pointwise_inverse(p.second.first, t)
    lhs.second.first = p.second.first(t).inverse
    rhs.second = Pair.new(p.second.first(t).inverse, p.second.second(t).inverse)
    lhs.second.first = rhs.second.first
    lhs.second.second = function_triple_inverse(p).second.second(t)
    function_triple_inverse(p).second.second = pointwise_inverse(p.second.second)
    lhs.second.second = pointwise_inverse(p.second.second, t)
    lhs.second.second = p.second.second(t).inverse
    lhs.second.second = rhs.second.second
    lhs.second = rhs.second
    pair_ext(lhs.second, rhs.second)
    pair_ext(lhs, rhs)
}

/// Pointwise addition is associative on a ternary product of function spaces.
theorem function_triple_add_assoc[T, A: AddSemigroup, B: AddSemigroup, C: AddSemigroup](
    p: Pair[T -> A, Pair[T -> B, T -> C]],
    q: Pair[T -> A, Pair[T -> B, T -> C]],
    r: Pair[T -> A, Pair[T -> B, T -> C]]
) {
    function_triple_add(p, function_triple_add(q, r)) =
    function_triple_add(function_triple_add(p, q), r)
} by {
    let lhs = function_triple_add(p, function_triple_add(q, r))
    let rhs = function_triple_add(function_triple_add(p, q), r)
    lhs.first = pointwise_add(p.first, function_triple_add(q, r).first)
    function_triple_add(q, r).first = pointwise_add(q.first, r.first)
    lhs.first = pointwise_add(p.first, pointwise_add(q.first, r.first))
    rhs.first = pointwise_add(function_triple_add(p, q).first, r.first)
    function_triple_add(p, q).first = pointwise_add(p.first, q.first)
    rhs.first = pointwise_add(pointwise_add(p.first, q.first), r.first)
    pointwise_add_assoc(p.first, q.first, r.first)
    lhs.first = rhs.first
    lhs.second.first = pointwise_add(p.second.first, function_triple_add(q, r).second.first)
    function_triple_add(q, r).second.first = pointwise_add(q.second.first, r.second.first)
    lhs.second.first = pointwise_add(p.second.first, pointwise_add(q.second.first, r.second.first))
    rhs.second.first = pointwise_add(function_triple_add(p, q).second.first, r.second.first)
    function_triple_add(p, q).second.first = pointwise_add(p.second.first, q.second.first)
    rhs.second.first = pointwise_add(pointwise_add(p.second.first, q.second.first), r.second.first)
    pointwise_add_assoc(p.second.first, q.second.first, r.second.first)
    lhs.second.first = rhs.second.first
    lhs.second.second = pointwise_add(p.second.second, function_triple_add(q, r).second.second)
    function_triple_add(q, r).second.second = pointwise_add(q.second.second, r.second.second)
    lhs.second.second = pointwise_add(p.second.second, pointwise_add(q.second.second, r.second.second))
    rhs.second.second = pointwise_add(function_triple_add(p, q).second.second, r.second.second)
    function_triple_add(p, q).second.second = pointwise_add(p.second.second, q.second.second)
    rhs.second.second = pointwise_add(pointwise_add(p.second.second, q.second.second), r.second.second)
    pointwise_add_assoc(p.second.second, q.second.second, r.second.second)
    lhs.second.second = rhs.second.second
    function_triple_ext(lhs, rhs)
}

/// Pointwise addition is commutative on a ternary product of function spaces.
theorem function_triple_add_comm[T, A: AddCommSemigroup, B: AddCommSemigroup, C: AddCommSemigroup](
    p: Pair[T -> A, Pair[T -> B, T -> C]],
    q: Pair[T -> A, Pair[T -> B, T -> C]]
) {
    function_triple_add(p, q) = function_triple_add(q, p)
} by {
    let lhs = function_triple_add(p, q)
    let rhs = function_triple_add(q, p)
    lhs.first = pointwise_add(p.first, q.first)
    rhs.first = pointwise_add(q.first, p.first)
    pointwise_add_comm(p.first, q.first)
    lhs.first = rhs.first
    lhs.second.first = pointwise_add(p.second.first, q.second.first)
    rhs.second.first = pointwise_add(q.second.first, p.second.first)
    pointwise_add_comm(p.second.first, q.second.first)
    lhs.second.first = rhs.second.first
    lhs.second.second = pointwise_add(p.second.second, q.second.second)
    rhs.second.second = pointwise_add(q.second.second, p.second.second)
    pointwise_add_comm(p.second.second, q.second.second)
    lhs.second.second = rhs.second.second
    function_triple_ext(lhs, rhs)
}

/// Adding the ternary pointwise zero on the right changes no coordinate.
theorem function_triple_add_zero_right[T, A: AddMonoid, B: AddMonoid, C: AddMonoid](
    p: Pair[T -> A, Pair[T -> B, T -> C]]
) {
    function_triple_add(p, function_triple_zero[T, A, B, C]) = p
} by {
    let lhs = function_triple_add(p, function_triple_zero[T, A, B, C])
    lhs.first = pointwise_add(p.first, function_triple_zero[T, A, B, C].first)
    function_triple_zero[T, A, B, C].first = pointwise_zero[T, A]
    lhs.first = pointwise_add(p.first, pointwise_zero[T, A])
    pointwise_add_zero_right(p.first)
    lhs.first = p.first
    lhs.second.first = pointwise_add(p.second.first, function_triple_zero[T, A, B, C].second.first)
    function_triple_zero[T, A, B, C].second.first = pointwise_zero[T, B]
    lhs.second.first = pointwise_add(p.second.first, pointwise_zero[T, B])
    pointwise_add_zero_right(p.second.first)
    lhs.second.first = p.second.first
    lhs.second.second = pointwise_add(p.second.second, function_triple_zero[T, A, B, C].second.second)
    function_triple_zero[T, A, B, C].second.second = pointwise_zero[T, C]
    lhs.second.second = pointwise_add(p.second.second, pointwise_zero[T, C])
    pointwise_add_zero_right(p.second.second)
    lhs.second.second = p.second.second
    function_triple_ext(lhs, p)
}

/// Adding the ternary pointwise zero on the left changes no coordinate.
theorem function_triple_add_zero_left[T, A: AddMonoid, B: AddMonoid, C: AddMonoid](
    p: Pair[T -> A, Pair[T -> B, T -> C]]
) {
    function_triple_add(function_triple_zero[T, A, B, C], p) = p
} by {
    let lhs = function_triple_add(function_triple_zero[T, A, B, C], p)
    lhs.first = pointwise_add(function_triple_zero[T, A, B, C].first, p.first)
    function_triple_zero[T, A, B, C].first = pointwise_zero[T, A]
    lhs.first = pointwise_add(pointwise_zero[T, A], p.first)
    pointwise_add_zero_left(p.first)
    lhs.first = p.first
    lhs.second.first = pointwise_add(function_triple_zero[T, A, B, C].second.first, p.second.first)
    function_triple_zero[T, A, B, C].second.first = pointwise_zero[T, B]
    lhs.second.first = pointwise_add(pointwise_zero[T, B], p.second.first)
    pointwise_add_zero_left(p.second.first)
    lhs.second.first = p.second.first
    lhs.second.second = pointwise_add(function_triple_zero[T, A, B, C].second.second, p.second.second)
    function_triple_zero[T, A, B, C].second.second = pointwise_zero[T, C]
    lhs.second.second = pointwise_add(pointwise_zero[T, C], p.second.second)
    pointwise_add_zero_left(p.second.second)
    lhs.second.second = p.second.second
    function_triple_ext(lhs, p)
}

/// Pointwise negation is a right additive inverse on a ternary product of function spaces.
theorem function_triple_add_neg_right[T, A: AddGroup, B: AddGroup, C: AddGroup](
    p: Pair[T -> A, Pair[T -> B, T -> C]]
) {
    function_triple_add(p, function_triple_neg(p)) = function_triple_zero[T, A, B, C]
} by {
    let lhs = function_triple_add(p, function_triple_neg(p))
    lhs.first = pointwise_add(p.first, function_triple_neg(p).first)
    function_triple_neg(p).first = pointwise_neg(p.first)
    lhs.first = pointwise_add(p.first, pointwise_neg(p.first))
    pointwise_add_neg_right(p.first)
    lhs.first = pointwise_zero[T, A]
    function_triple_zero[T, A, B, C].first = pointwise_zero[T, A]
    lhs.second.first = pointwise_add(p.second.first, function_triple_neg(p).second.first)
    function_triple_neg(p).second.first = pointwise_neg(p.second.first)
    lhs.second.first = pointwise_add(p.second.first, pointwise_neg(p.second.first))
    pointwise_add_neg_right(p.second.first)
    lhs.second.first = pointwise_zero[T, B]
    function_triple_zero[T, A, B, C].second.first = pointwise_zero[T, B]
    lhs.second.second = pointwise_add(p.second.second, function_triple_neg(p).second.second)
    function_triple_neg(p).second.second = pointwise_neg(p.second.second)
    lhs.second.second = pointwise_add(p.second.second, pointwise_neg(p.second.second))
    pointwise_add_neg_right(p.second.second)
    lhs.second.second = pointwise_zero[T, C]
    function_triple_zero[T, A, B, C].second.second = pointwise_zero[T, C]
    function_triple_ext(lhs, function_triple_zero[T, A, B, C])
}

/// Pointwise negation is a left additive inverse on a ternary product of function spaces.
theorem function_triple_add_neg_left[T, A: AddGroup, B: AddGroup, C: AddGroup](
    p: Pair[T -> A, Pair[T -> B, T -> C]]
) {
    function_triple_add(function_triple_neg(p), p) = function_triple_zero[T, A, B, C]
} by {
    let lhs = function_triple_add(function_triple_neg(p), p)
    lhs.first = pointwise_add(function_triple_neg(p).first, p.first)
    function_triple_neg(p).first = pointwise_neg(p.first)
    lhs.first = pointwise_add(pointwise_neg(p.first), p.first)
    pointwise_add_neg_left(p.first)
    lhs.first = pointwise_zero[T, A]
    function_triple_zero[T, A, B, C].first = pointwise_zero[T, A]
    lhs.second.first = pointwise_add(function_triple_neg(p).second.first, p.second.first)
    function_triple_neg(p).second.first = pointwise_neg(p.second.first)
    lhs.second.first = pointwise_add(pointwise_neg(p.second.first), p.second.first)
    pointwise_add_neg_left(p.second.first)
    lhs.second.first = pointwise_zero[T, B]
    function_triple_zero[T, A, B, C].second.first = pointwise_zero[T, B]
    lhs.second.second = pointwise_add(function_triple_neg(p).second.second, p.second.second)
    function_triple_neg(p).second.second = pointwise_neg(p.second.second)
    lhs.second.second = pointwise_add(pointwise_neg(p.second.second), p.second.second)
    pointwise_add_neg_left(p.second.second)
    lhs.second.second = pointwise_zero[T, C]
    function_triple_zero[T, A, B, C].second.second = pointwise_zero[T, C]
    function_triple_ext(lhs, function_triple_zero[T, A, B, C])
}

/// Pointwise multiplication is associative on a ternary product of function spaces.
theorem function_triple_mul_assoc[T, A: Semigroup, B: Semigroup, C: Semigroup](
    p: Pair[T -> A, Pair[T -> B, T -> C]],
    q: Pair[T -> A, Pair[T -> B, T -> C]],
    r: Pair[T -> A, Pair[T -> B, T -> C]]
) {
    function_triple_mul(p, function_triple_mul(q, r)) =
    function_triple_mul(function_triple_mul(p, q), r)
} by {
    let lhs = function_triple_mul(p, function_triple_mul(q, r))
    let rhs = function_triple_mul(function_triple_mul(p, q), r)
    lhs.first = pointwise_mul(p.first, function_triple_mul(q, r).first)
    function_triple_mul(q, r).first = pointwise_mul(q.first, r.first)
    lhs.first = pointwise_mul(p.first, pointwise_mul(q.first, r.first))
    rhs.first = pointwise_mul(function_triple_mul(p, q).first, r.first)
    function_triple_mul(p, q).first = pointwise_mul(p.first, q.first)
    rhs.first = pointwise_mul(pointwise_mul(p.first, q.first), r.first)
    pointwise_mul_assoc(p.first, q.first, r.first)
    lhs.first = rhs.first
    lhs.second.first = pointwise_mul(p.second.first, function_triple_mul(q, r).second.first)
    function_triple_mul(q, r).second.first = pointwise_mul(q.second.first, r.second.first)
    lhs.second.first = pointwise_mul(p.second.first, pointwise_mul(q.second.first, r.second.first))
    rhs.second.first = pointwise_mul(function_triple_mul(p, q).second.first, r.second.first)
    function_triple_mul(p, q).second.first = pointwise_mul(p.second.first, q.second.first)
    rhs.second.first = pointwise_mul(pointwise_mul(p.second.first, q.second.first), r.second.first)
    pointwise_mul_assoc(p.second.first, q.second.first, r.second.first)
    lhs.second.first = rhs.second.first
    lhs.second.second = pointwise_mul(p.second.second, function_triple_mul(q, r).second.second)
    function_triple_mul(q, r).second.second = pointwise_mul(q.second.second, r.second.second)
    lhs.second.second = pointwise_mul(p.second.second, pointwise_mul(q.second.second, r.second.second))
    rhs.second.second = pointwise_mul(function_triple_mul(p, q).second.second, r.second.second)
    function_triple_mul(p, q).second.second = pointwise_mul(p.second.second, q.second.second)
    rhs.second.second = pointwise_mul(pointwise_mul(p.second.second, q.second.second), r.second.second)
    pointwise_mul_assoc(p.second.second, q.second.second, r.second.second)
    lhs.second.second = rhs.second.second
    function_triple_ext(lhs, rhs)
}

/// Pointwise multiplication is commutative on a ternary product of function spaces.
theorem function_triple_mul_comm[T, A: CommSemigroup, B: CommSemigroup, C: CommSemigroup](
    p: Pair[T -> A, Pair[T -> B, T -> C]],
    q: Pair[T -> A, Pair[T -> B, T -> C]]
) {
    function_triple_mul(p, q) = function_triple_mul(q, p)
} by {
    let lhs = function_triple_mul(p, q)
    let rhs = function_triple_mul(q, p)
    lhs.first = pointwise_mul(p.first, q.first)
    rhs.first = pointwise_mul(q.first, p.first)
    pointwise_mul_comm(p.first, q.first)
    lhs.first = rhs.first
    lhs.second.first = pointwise_mul(p.second.first, q.second.first)
    rhs.second.first = pointwise_mul(q.second.first, p.second.first)
    pointwise_mul_comm(p.second.first, q.second.first)
    lhs.second.first = rhs.second.first
    lhs.second.second = pointwise_mul(p.second.second, q.second.second)
    rhs.second.second = pointwise_mul(q.second.second, p.second.second)
    pointwise_mul_comm(p.second.second, q.second.second)
    lhs.second.second = rhs.second.second
    function_triple_ext(lhs, rhs)
}

/// Multiplying by the ternary pointwise one on the right changes no coordinate.
theorem function_triple_mul_one_right[T, A: Monoid, B: Monoid, C: Monoid](
    p: Pair[T -> A, Pair[T -> B, T -> C]]
) {
    function_triple_mul(p, function_triple_one[T, A, B, C]) = p
} by {
    let lhs = function_triple_mul(p, function_triple_one[T, A, B, C])
    lhs.first = pointwise_mul(p.first, function_triple_one[T, A, B, C].first)
    function_triple_one[T, A, B, C].first = pointwise_one[T, A]
    lhs.first = pointwise_mul(p.first, pointwise_one[T, A])
    pointwise_mul_one_right(p.first)
    lhs.first = p.first
    lhs.second.first = pointwise_mul(p.second.first, function_triple_one[T, A, B, C].second.first)
    function_triple_one[T, A, B, C].second.first = pointwise_one[T, B]
    lhs.second.first = pointwise_mul(p.second.first, pointwise_one[T, B])
    pointwise_mul_one_right(p.second.first)
    lhs.second.first = p.second.first
    lhs.second.second = pointwise_mul(p.second.second, function_triple_one[T, A, B, C].second.second)
    function_triple_one[T, A, B, C].second.second = pointwise_one[T, C]
    lhs.second.second = pointwise_mul(p.second.second, pointwise_one[T, C])
    pointwise_mul_one_right(p.second.second)
    lhs.second.second = p.second.second
    function_triple_ext(lhs, p)
}

/// Multiplying by the ternary pointwise one on the left changes no coordinate.
theorem function_triple_mul_one_left[T, A: Monoid, B: Monoid, C: Monoid](
    p: Pair[T -> A, Pair[T -> B, T -> C]]
) {
    function_triple_mul(function_triple_one[T, A, B, C], p) = p
} by {
    let lhs = function_triple_mul(function_triple_one[T, A, B, C], p)
    lhs.first = pointwise_mul(function_triple_one[T, A, B, C].first, p.first)
    function_triple_one[T, A, B, C].first = pointwise_one[T, A]
    lhs.first = pointwise_mul(pointwise_one[T, A], p.first)
    pointwise_mul_one_left(p.first)
    lhs.first = p.first
    lhs.second.first = pointwise_mul(function_triple_one[T, A, B, C].second.first, p.second.first)
    function_triple_one[T, A, B, C].second.first = pointwise_one[T, B]
    lhs.second.first = pointwise_mul(pointwise_one[T, B], p.second.first)
    pointwise_mul_one_left(p.second.first)
    lhs.second.first = p.second.first
    lhs.second.second = pointwise_mul(function_triple_one[T, A, B, C].second.second, p.second.second)
    function_triple_one[T, A, B, C].second.second = pointwise_one[T, C]
    lhs.second.second = pointwise_mul(pointwise_one[T, C], p.second.second)
    pointwise_mul_one_left(p.second.second)
    lhs.second.second = p.second.second
    function_triple_ext(lhs, p)
}

/// Pointwise inversion is a right inverse on a ternary product of function spaces.
theorem function_triple_mul_inverse_right[T, A: Group, B: Group, C: Group](
    p: Pair[T -> A, Pair[T -> B, T -> C]]
) {
    function_triple_mul(p, function_triple_inverse(p)) = function_triple_one[T, A, B, C]
} by {
    let lhs = function_triple_mul(p, function_triple_inverse(p))
    lhs.first = pointwise_mul(p.first, function_triple_inverse(p).first)
    function_triple_inverse(p).first = pointwise_inverse(p.first)
    lhs.first = pointwise_mul(p.first, pointwise_inverse(p.first))
    pointwise_mul_inverse_right(p.first)
    lhs.first = pointwise_one[T, A]
    function_triple_one[T, A, B, C].first = pointwise_one[T, A]
    lhs.second.first = pointwise_mul(p.second.first, function_triple_inverse(p).second.first)
    function_triple_inverse(p).second.first = pointwise_inverse(p.second.first)
    lhs.second.first = pointwise_mul(p.second.first, pointwise_inverse(p.second.first))
    pointwise_mul_inverse_right(p.second.first)
    lhs.second.first = pointwise_one[T, B]
    function_triple_one[T, A, B, C].second.first = pointwise_one[T, B]
    lhs.second.second = pointwise_mul(p.second.second, function_triple_inverse(p).second.second)
    function_triple_inverse(p).second.second = pointwise_inverse(p.second.second)
    lhs.second.second = pointwise_mul(p.second.second, pointwise_inverse(p.second.second))
    pointwise_mul_inverse_right(p.second.second)
    lhs.second.second = pointwise_one[T, C]
    function_triple_one[T, A, B, C].second.second = pointwise_one[T, C]
    function_triple_ext(lhs, function_triple_one[T, A, B, C])
}

/// Pointwise inversion is a left inverse on a ternary product of function spaces.
theorem function_triple_mul_inverse_left[T, A: Group, B: Group, C: Group](
    p: Pair[T -> A, Pair[T -> B, T -> C]]
) {
    function_triple_mul(function_triple_inverse(p), p) = function_triple_one[T, A, B, C]
} by {
    let lhs = function_triple_mul(function_triple_inverse(p), p)
    lhs.first = pointwise_mul(function_triple_inverse(p).first, p.first)
    function_triple_inverse(p).first = pointwise_inverse(p.first)
    lhs.first = pointwise_mul(pointwise_inverse(p.first), p.first)
    pointwise_mul_inverse_left(p.first)
    lhs.first = pointwise_one[T, A]
    function_triple_one[T, A, B, C].first = pointwise_one[T, A]
    lhs.second.first = pointwise_mul(function_triple_inverse(p).second.first, p.second.first)
    function_triple_inverse(p).second.first = pointwise_inverse(p.second.first)
    lhs.second.first = pointwise_mul(pointwise_inverse(p.second.first), p.second.first)
    pointwise_mul_inverse_left(p.second.first)
    lhs.second.first = pointwise_one[T, B]
    function_triple_one[T, A, B, C].second.first = pointwise_one[T, B]
    lhs.second.second = pointwise_mul(function_triple_inverse(p).second.second, p.second.second)
    function_triple_inverse(p).second.second = pointwise_inverse(p.second.second)
    lhs.second.second = pointwise_mul(pointwise_inverse(p.second.second), p.second.second)
    pointwise_mul_inverse_left(p.second.second)
    lhs.second.second = pointwise_one[T, C]
    function_triple_one[T, A, B, C].second.second = pointwise_one[T, C]
    function_triple_ext(lhs, function_triple_one[T, A, B, C])
}

/// Pointwise multiplication distributes over ternary pointwise addition on the left.
theorem function_triple_mul_distrib_left[T, A: Semiring, B: Semiring, C: Semiring](
    p: Pair[T -> A, Pair[T -> B, T -> C]],
    q: Pair[T -> A, Pair[T -> B, T -> C]],
    r: Pair[T -> A, Pair[T -> B, T -> C]]
) {
    function_triple_mul(p, function_triple_add(q, r)) =
    function_triple_add(function_triple_mul(p, q), function_triple_mul(p, r))
} by {
    let lhs = function_triple_mul(p, function_triple_add(q, r))
    let rhs = function_triple_add(function_triple_mul(p, q), function_triple_mul(p, r))
    lhs.first = pointwise_mul(p.first, function_triple_add(q, r).first)
    function_triple_add(q, r).first = pointwise_add(q.first, r.first)
    lhs.first = pointwise_mul(p.first, pointwise_add(q.first, r.first))
    rhs.first = pointwise_add(function_triple_mul(p, q).first, function_triple_mul(p, r).first)
    function_triple_mul(p, q).first = pointwise_mul(p.first, q.first)
    function_triple_mul(p, r).first = pointwise_mul(p.first, r.first)
    rhs.first = pointwise_add(pointwise_mul(p.first, q.first), pointwise_mul(p.first, r.first))
    pointwise_mul_distrib_left(p.first, q.first, r.first)
    lhs.first = rhs.first
    lhs.second.first = pointwise_mul(p.second.first, function_triple_add(q, r).second.first)
    function_triple_add(q, r).second.first = pointwise_add(q.second.first, r.second.first)
    lhs.second.first = pointwise_mul(p.second.first, pointwise_add(q.second.first, r.second.first))
    rhs.second.first = pointwise_add(
        function_triple_mul(p, q).second.first,
        function_triple_mul(p, r).second.first)
    function_triple_mul(p, q).second.first = pointwise_mul(p.second.first, q.second.first)
    function_triple_mul(p, r).second.first = pointwise_mul(p.second.first, r.second.first)
    rhs.second.first =
        pointwise_add(pointwise_mul(p.second.first, q.second.first),
            pointwise_mul(p.second.first, r.second.first))
    pointwise_mul_distrib_left(p.second.first, q.second.first, r.second.first)
    lhs.second.first = rhs.second.first
    lhs.second.second = pointwise_mul(p.second.second, function_triple_add(q, r).second.second)
    function_triple_add(q, r).second.second = pointwise_add(q.second.second, r.second.second)
    lhs.second.second = pointwise_mul(p.second.second,
        pointwise_add(q.second.second, r.second.second))
    rhs.second.second = pointwise_add(
        function_triple_mul(p, q).second.second,
        function_triple_mul(p, r).second.second)
    function_triple_mul(p, q).second.second = pointwise_mul(p.second.second, q.second.second)
    function_triple_mul(p, r).second.second = pointwise_mul(p.second.second, r.second.second)
    rhs.second.second =
        pointwise_add(pointwise_mul(p.second.second, q.second.second),
            pointwise_mul(p.second.second, r.second.second))
    pointwise_mul_distrib_left(p.second.second, q.second.second, r.second.second)
    lhs.second.second = rhs.second.second
    function_triple_ext(lhs, rhs)
}

/// Pointwise multiplication distributes over ternary pointwise addition on the right.
theorem function_triple_mul_distrib_right[T, A: Semiring, B: Semiring, C: Semiring](
    p: Pair[T -> A, Pair[T -> B, T -> C]],
    q: Pair[T -> A, Pair[T -> B, T -> C]],
    r: Pair[T -> A, Pair[T -> B, T -> C]]
) {
    function_triple_mul(function_triple_add(p, q), r) =
    function_triple_add(function_triple_mul(p, r), function_triple_mul(q, r))
} by {
    let lhs = function_triple_mul(function_triple_add(p, q), r)
    let rhs = function_triple_add(function_triple_mul(p, r), function_triple_mul(q, r))
    lhs.first = pointwise_mul(function_triple_add(p, q).first, r.first)
    function_triple_add(p, q).first = pointwise_add(p.first, q.first)
    lhs.first = pointwise_mul(pointwise_add(p.first, q.first), r.first)
    rhs.first = pointwise_add(function_triple_mul(p, r).first, function_triple_mul(q, r).first)
    function_triple_mul(p, r).first = pointwise_mul(p.first, r.first)
    function_triple_mul(q, r).first = pointwise_mul(q.first, r.first)
    rhs.first = pointwise_add(pointwise_mul(p.first, r.first), pointwise_mul(q.first, r.first))
    pointwise_mul_distrib_right(p.first, q.first, r.first)
    lhs.first = rhs.first
    lhs.second.first = pointwise_mul(function_triple_add(p, q).second.first, r.second.first)
    function_triple_add(p, q).second.first = pointwise_add(p.second.first, q.second.first)
    lhs.second.first = pointwise_mul(pointwise_add(p.second.first, q.second.first), r.second.first)
    rhs.second.first = pointwise_add(
        function_triple_mul(p, r).second.first,
        function_triple_mul(q, r).second.first)
    function_triple_mul(p, r).second.first = pointwise_mul(p.second.first, r.second.first)
    function_triple_mul(q, r).second.first = pointwise_mul(q.second.first, r.second.first)
    rhs.second.first =
        pointwise_add(pointwise_mul(p.second.first, r.second.first),
            pointwise_mul(q.second.first, r.second.first))
    pointwise_mul_distrib_right(p.second.first, q.second.first, r.second.first)
    lhs.second.first = rhs.second.first
    lhs.second.second = pointwise_mul(function_triple_add(p, q).second.second, r.second.second)
    function_triple_add(p, q).second.second = pointwise_add(p.second.second, q.second.second)
    lhs.second.second =
        pointwise_mul(pointwise_add(p.second.second, q.second.second), r.second.second)
    rhs.second.second = pointwise_add(
        function_triple_mul(p, r).second.second,
        function_triple_mul(q, r).second.second)
    function_triple_mul(p, r).second.second = pointwise_mul(p.second.second, r.second.second)
    function_triple_mul(q, r).second.second = pointwise_mul(q.second.second, r.second.second)
    rhs.second.second =
        pointwise_add(pointwise_mul(p.second.second, r.second.second),
            pointwise_mul(q.second.second, r.second.second))
    pointwise_mul_distrib_right(p.second.second, q.second.second, r.second.second)
    lhs.second.second = rhs.second.second
    function_triple_ext(lhs, rhs)
}

/// Multiplying by the ternary pointwise zero on the right gives the pointwise zero.
theorem function_triple_mul_zero_right[T, A: Semiring, B: Semiring, C: Semiring](
    p: Pair[T -> A, Pair[T -> B, T -> C]]
) {
    function_triple_mul(p, function_triple_zero[T, A, B, C]) = function_triple_zero[T, A, B, C]
} by {
    let lhs = function_triple_mul(p, function_triple_zero[T, A, B, C])
    lhs.first = pointwise_mul(p.first, function_triple_zero[T, A, B, C].first)
    function_triple_zero[T, A, B, C].first = pointwise_zero[T, A]
    lhs.first = pointwise_mul(p.first, pointwise_zero[T, A])
    pointwise_mul_zero_right(p.first)
    lhs.first = pointwise_zero[T, A]
    lhs.second.first = pointwise_mul(p.second.first, function_triple_zero[T, A, B, C].second.first)
    function_triple_zero[T, A, B, C].second.first = pointwise_zero[T, B]
    lhs.second.first = pointwise_mul(p.second.first, pointwise_zero[T, B])
    pointwise_mul_zero_right(p.second.first)
    lhs.second.first = pointwise_zero[T, B]
    lhs.second.second = pointwise_mul(p.second.second, function_triple_zero[T, A, B, C].second.second)
    function_triple_zero[T, A, B, C].second.second = pointwise_zero[T, C]
    lhs.second.second = pointwise_mul(p.second.second, pointwise_zero[T, C])
    pointwise_mul_zero_right(p.second.second)
    lhs.second.second = pointwise_zero[T, C]
    function_triple_ext(lhs, function_triple_zero[T, A, B, C])
}

/// Multiplying by the ternary pointwise zero on the left gives the pointwise zero.
theorem function_triple_mul_zero_left[T, A: Semiring, B: Semiring, C: Semiring](
    p: Pair[T -> A, Pair[T -> B, T -> C]]
) {
    function_triple_mul(function_triple_zero[T, A, B, C], p) = function_triple_zero[T, A, B, C]
} by {
    let lhs = function_triple_mul(function_triple_zero[T, A, B, C], p)
    lhs.first = pointwise_mul(function_triple_zero[T, A, B, C].first, p.first)
    function_triple_zero[T, A, B, C].first = pointwise_zero[T, A]
    lhs.first = pointwise_mul(pointwise_zero[T, A], p.first)
    pointwise_mul_zero_left(p.first)
    lhs.first = pointwise_zero[T, A]
    lhs.second.first = pointwise_mul(function_triple_zero[T, A, B, C].second.first, p.second.first)
    function_triple_zero[T, A, B, C].second.first = pointwise_zero[T, B]
    lhs.second.first = pointwise_mul(pointwise_zero[T, B], p.second.first)
    pointwise_mul_zero_left(p.second.first)
    lhs.second.first = pointwise_zero[T, B]
    lhs.second.second = pointwise_mul(function_triple_zero[T, A, B, C].second.second, p.second.second)
    function_triple_zero[T, A, B, C].second.second = pointwise_zero[T, C]
    lhs.second.second = pointwise_mul(pointwise_zero[T, C], p.second.second)
    pointwise_mul_zero_left(p.second.second)
    lhs.second.second = pointwise_zero[T, C]
    function_triple_ext(lhs, function_triple_zero[T, A, B, C])
}

/// The value of a quaternary product of functions at one point.
define function_quadruple_apply[T, A, B, C, D](
    p: Pair[T -> A, Pair[T -> B, Pair[T -> C, T -> D]]],
    t: T
) -> Pair[A, Pair[B, Pair[C, D]]] {
    Pair.new(p.first(t),
        Pair.new(p.second.first(t),
            Pair.new(p.second.second.first(t), p.second.second.second(t))))
}

/// The quaternary product of four functions.
define function_quadruple[T, A, B, C, D](
    f: T -> A,
    g: T -> B,
    h: T -> C,
    i: T -> D
) -> Pair[T -> A, Pair[T -> B, Pair[T -> C, T -> D]]] {
    Pair.new(f, Pair.new(g, Pair.new(h, i)))
}

/// The pointwise sum on a quaternary product of function spaces.
define function_quadruple_add[T, A: Add, B: Add, C: Add, D: Add](
    p: Pair[T -> A, Pair[T -> B, Pair[T -> C, T -> D]]],
    q: Pair[T -> A, Pair[T -> B, Pair[T -> C, T -> D]]]
) -> Pair[T -> A, Pair[T -> B, Pair[T -> C, T -> D]]] {
    Pair.new(pointwise_add(p.first, q.first),
        Pair.new(pointwise_add(p.second.first, q.second.first),
            Pair.new(pointwise_add(p.second.second.first, q.second.second.first),
                pointwise_add(p.second.second.second, q.second.second.second))))
}

/// The pointwise product on a quaternary product of function spaces.
define function_quadruple_mul[T, A: Mul, B: Mul, C: Mul, D: Mul](
    p: Pair[T -> A, Pair[T -> B, Pair[T -> C, T -> D]]],
    q: Pair[T -> A, Pair[T -> B, Pair[T -> C, T -> D]]]
) -> Pair[T -> A, Pair[T -> B, Pair[T -> C, T -> D]]] {
    Pair.new(pointwise_mul(p.first, q.first),
        Pair.new(pointwise_mul(p.second.first, q.second.first),
            Pair.new(pointwise_mul(p.second.second.first, q.second.second.first),
                pointwise_mul(p.second.second.second, q.second.second.second))))
}

/// The pointwise zero on a quaternary product of function spaces.
let function_quadruple_zero[T, A: Zero, B: Zero, C: Zero, D: Zero]:
    Pair[T -> A, Pair[T -> B, Pair[T -> C, T -> D]]] =
    Pair.new(pointwise_zero[T, A],
        Pair.new(pointwise_zero[T, B],
            Pair.new(pointwise_zero[T, C], pointwise_zero[T, D])))

/// The pointwise one on a quaternary product of function spaces.
let function_quadruple_one[T, A: One, B: One, C: One, D: One]:
    Pair[T -> A, Pair[T -> B, Pair[T -> C, T -> D]]] =
    Pair.new(pointwise_one[T, A],
        Pair.new(pointwise_one[T, B],
            Pair.new(pointwise_one[T, C], pointwise_one[T, D])))

/// The pointwise negation on a quaternary product of function spaces.
define function_quadruple_neg[T, A: Neg, B: Neg, C: Neg, D: Neg](
    p: Pair[T -> A, Pair[T -> B, Pair[T -> C, T -> D]]]
) -> Pair[T -> A, Pair[T -> B, Pair[T -> C, T -> D]]] {
    Pair.new(pointwise_neg(p.first),
        Pair.new(pointwise_neg(p.second.first),
            Pair.new(pointwise_neg(p.second.second.first),
                pointwise_neg(p.second.second.second))))
}

/// The pointwise inverse on a quaternary product of function spaces.
define function_quadruple_inverse[T, A: Group, B: Group, C: Group, D: Group](
    p: Pair[T -> A, Pair[T -> B, Pair[T -> C, T -> D]]]
) -> Pair[T -> A, Pair[T -> B, Pair[T -> C, T -> D]]] {
    Pair.new(pointwise_inverse(p.first),
        Pair.new(pointwise_inverse(p.second.first),
            Pair.new(pointwise_inverse(p.second.second.first),
                pointwise_inverse(p.second.second.second))))
}

/// Evaluation of a quaternary product has the expected first coordinate.
theorem function_quadruple_apply_first[T, A, B, C, D](
    p: Pair[T -> A, Pair[T -> B, Pair[T -> C, T -> D]]],
    t: T
) {
    function_quadruple_apply(p, t).first = p.first(t)
} by {
    function_quadruple_apply(p, t) = Pair.new(p.first(t),
        Pair.new(p.second.first(t),
            Pair.new(p.second.second.first(t), p.second.second.second(t))))
}

/// Evaluation of a quaternary product has the expected second coordinate.
theorem function_quadruple_apply_second_first[T, A, B, C, D](
    p: Pair[T -> A, Pair[T -> B, Pair[T -> C, T -> D]]],
    t: T
) {
    function_quadruple_apply(p, t).second.first = p.second.first(t)
} by {
    function_quadruple_apply(p, t).second = Pair.new(p.second.first(t),
        Pair.new(p.second.second.first(t), p.second.second.second(t)))
}

/// Evaluation of a quaternary product has the expected third coordinate.
theorem function_quadruple_apply_second_second_first[T, A, B, C, D](
    p: Pair[T -> A, Pair[T -> B, Pair[T -> C, T -> D]]],
    t: T
) {
    function_quadruple_apply(p, t).second.second.first = p.second.second.first(t)
} by {
    let lhs = function_quadruple_apply(p, t)
    lhs = Pair.new(p.first(t),
        Pair.new(p.second.first(t),
            Pair.new(p.second.second.first(t), p.second.second.second(t))))
    lhs.second = Pair.new(p.second.first(t),
        Pair.new(p.second.second.first(t), p.second.second.second(t)))
    lhs.second.second = Pair.new(p.second.second.first(t), p.second.second.second(t))
}

/// Evaluation of a quaternary product has the expected fourth coordinate.
theorem function_quadruple_apply_second_second_second[T, A, B, C, D](
    p: Pair[T -> A, Pair[T -> B, Pair[T -> C, T -> D]]],
    t: T
) {
    function_quadruple_apply(p, t).second.second.second = p.second.second.second(t)
} by {
    let lhs = function_quadruple_apply(p, t)
    lhs = Pair.new(p.first(t),
        Pair.new(p.second.first(t),
            Pair.new(p.second.second.first(t), p.second.second.second(t))))
    lhs.second = Pair.new(p.second.first(t),
        Pair.new(p.second.second.first(t), p.second.second.second(t)))
    lhs.second.second = Pair.new(p.second.second.first(t), p.second.second.second(t))
}

/// The quaternary product of four functions has the expected first coordinate.
theorem function_quadruple_first[T, A, B, C, D](
    f: T -> A, g: T -> B, h: T -> C, i: T -> D
) {
    function_quadruple(f, g, h, i).first = f
}

/// The quaternary product of four functions has the expected second coordinate.
theorem function_quadruple_second_first[T, A, B, C, D](
    f: T -> A, g: T -> B, h: T -> C, i: T -> D
) {
    function_quadruple(f, g, h, i).second.first = g
}

/// The quaternary product of four functions has the expected third coordinate.
theorem function_quadruple_second_second_first[T, A, B, C, D](
    f: T -> A, g: T -> B, h: T -> C, i: T -> D
) {
    function_quadruple(f, g, h, i).second.second.first = h
}

/// The quaternary product of four functions has the expected fourth coordinate.
theorem function_quadruple_second_second_second[T, A, B, C, D](
    f: T -> A, g: T -> B, h: T -> C, i: T -> D
) {
    function_quadruple(f, g, h, i).second.second.second = i
}

/// Function quadruples are determined by their four coordinates.
theorem function_quadruple_eta[T, A, B, C, D](
    p: Pair[T -> A, Pair[T -> B, Pair[T -> C, T -> D]]]
) {
    function_quadruple(p.first, p.second.first, p.second.second.first,
        p.second.second.second) = p
} by {
    let q = function_quadruple(p.first, p.second.first, p.second.second.first,
        p.second.second.second)
    q.first = p.first
    q.second.first = p.second.first
    q.second.second.first = p.second.second.first
    q.second.second.second = p.second.second.second
    q.second.second = p.second.second
    pair_ext(q.second.second, p.second.second)
    q.second = p.second
    pair_ext(q.second, p.second)
    pair_ext(q, p)
}

/// Quaternary function products are determined by their four coordinates.
theorem function_quadruple_ext[T, A, B, C, D](
    p: Pair[T -> A, Pair[T -> B, Pair[T -> C, T -> D]]],
    q: Pair[T -> A, Pair[T -> B, Pair[T -> C, T -> D]]]
) {
    p.first = q.first and p.second.first = q.second.first and
    p.second.second.first = q.second.second.first and
    p.second.second.second = q.second.second.second implies p = q
} by {
    if p.first = q.first and p.second.first = q.second.first and
    p.second.second.first = q.second.second.first and
    p.second.second.second = q.second.second.second {
        p.second.second = q.second.second
        pair_ext(p.second.second, q.second.second)
        p.second = q.second
        pair_ext(p.second, q.second)
        pair_ext(p, q)
    }
}

/// The first coordinate of a quaternary pointwise sum is the pointwise sum of first coordinates.
theorem function_quadruple_add_first[T, A: Add, B: Add, C: Add, D: Add](
    p: Pair[T -> A, Pair[T -> B, Pair[T -> C, T -> D]]],
    q: Pair[T -> A, Pair[T -> B, Pair[T -> C, T -> D]]]
) {
    function_quadruple_add(p, q).first = pointwise_add(p.first, q.first)
} by {
    function_quadruple_add(p, q) = Pair.new(pointwise_add(p.first, q.first),
        Pair.new(pointwise_add(p.second.first, q.second.first),
            Pair.new(pointwise_add(p.second.second.first, q.second.second.first),
                pointwise_add(p.second.second.second, q.second.second.second))))
}

/// The second coordinate of a quaternary pointwise sum is the pointwise sum of second coordinates.
theorem function_quadruple_add_second_first[T, A: Add, B: Add, C: Add, D: Add](
    p: Pair[T -> A, Pair[T -> B, Pair[T -> C, T -> D]]],
    q: Pair[T -> A, Pair[T -> B, Pair[T -> C, T -> D]]]
) {
    function_quadruple_add(p, q).second.first =
        pointwise_add(p.second.first, q.second.first)
} by {
    let inner = Pair.new(pointwise_add(p.second.first, q.second.first),
        Pair.new(pointwise_add(p.second.second.first, q.second.second.first),
            pointwise_add(p.second.second.second, q.second.second.second)))
    inner.first = pointwise_add(p.second.first, q.second.first)
    function_quadruple_add(p, q) = Pair.new(pointwise_add(p.first, q.first), inner)
    function_quadruple_add(p, q).second = inner
}

/// The third coordinate of a quaternary pointwise sum is the pointwise sum of third coordinates.
theorem function_quadruple_add_second_second_first[T, A: Add, B: Add, C: Add, D: Add](
    p: Pair[T -> A, Pair[T -> B, Pair[T -> C, T -> D]]],
    q: Pair[T -> A, Pair[T -> B, Pair[T -> C, T -> D]]]
) {
    function_quadruple_add(p, q).second.second.first =
        pointwise_add(p.second.second.first, q.second.second.first)
} by {
    function_quadruple_add(p, q) = Pair.new(pointwise_add(p.first, q.first),
        Pair.new(pointwise_add(p.second.first, q.second.first),
            Pair.new(pointwise_add(p.second.second.first, q.second.second.first),
                pointwise_add(p.second.second.second, q.second.second.second))))
    function_quadruple_add(p, q).second =
        Pair.new(pointwise_add(p.second.first, q.second.first),
            Pair.new(pointwise_add(p.second.second.first, q.second.second.first),
                pointwise_add(p.second.second.second, q.second.second.second)))
    function_quadruple_add(p, q).second.second =
        Pair.new(pointwise_add(p.second.second.first, q.second.second.first),
            pointwise_add(p.second.second.second, q.second.second.second))
}

/// The fourth coordinate of a quaternary pointwise sum is the pointwise sum of fourth coordinates.
theorem function_quadruple_add_second_second_second[T, A: Add, B: Add, C: Add, D: Add](
    p: Pair[T -> A, Pair[T -> B, Pair[T -> C, T -> D]]],
    q: Pair[T -> A, Pair[T -> B, Pair[T -> C, T -> D]]]
) {
    function_quadruple_add(p, q).second.second.second =
        pointwise_add(p.second.second.second, q.second.second.second)
} by {
    function_quadruple_add(p, q) = Pair.new(pointwise_add(p.first, q.first),
        Pair.new(pointwise_add(p.second.first, q.second.first),
            Pair.new(pointwise_add(p.second.second.first, q.second.second.first),
                pointwise_add(p.second.second.second, q.second.second.second))))
    function_quadruple_add(p, q).second =
        Pair.new(pointwise_add(p.second.first, q.second.first),
            Pair.new(pointwise_add(p.second.second.first, q.second.second.first),
                pointwise_add(p.second.second.second, q.second.second.second)))
    function_quadruple_add(p, q).second.second =
        Pair.new(pointwise_add(p.second.second.first, q.second.second.first),
            pointwise_add(p.second.second.second, q.second.second.second))
}

/// The first coordinate of a quaternary pointwise product is the pointwise product of first coordinates.
theorem function_quadruple_mul_first[T, A: Mul, B: Mul, C: Mul, D: Mul](
    p: Pair[T -> A, Pair[T -> B, Pair[T -> C, T -> D]]],
    q: Pair[T -> A, Pair[T -> B, Pair[T -> C, T -> D]]]
) {
    function_quadruple_mul(p, q).first = pointwise_mul(p.first, q.first)
} by {
    function_quadruple_mul(p, q) = Pair.new(pointwise_mul(p.first, q.first),
        Pair.new(pointwise_mul(p.second.first, q.second.first),
            Pair.new(pointwise_mul(p.second.second.first, q.second.second.first),
                pointwise_mul(p.second.second.second, q.second.second.second))))
}

/// The second coordinate of a quaternary pointwise product is the pointwise product of second coordinates.
theorem function_quadruple_mul_second_first[T, A: Mul, B: Mul, C: Mul, D: Mul](
    p: Pair[T -> A, Pair[T -> B, Pair[T -> C, T -> D]]],
    q: Pair[T -> A, Pair[T -> B, Pair[T -> C, T -> D]]]
) {
    function_quadruple_mul(p, q).second.first =
        pointwise_mul(p.second.first, q.second.first)
} by {
    let inner = Pair.new(pointwise_mul(p.second.first, q.second.first),
        Pair.new(pointwise_mul(p.second.second.first, q.second.second.first),
            pointwise_mul(p.second.second.second, q.second.second.second)))
    inner.first = pointwise_mul(p.second.first, q.second.first)
    function_quadruple_mul(p, q) = Pair.new(pointwise_mul(p.first, q.first), inner)
    function_quadruple_mul(p, q).second = inner
}

/// The third coordinate of a quaternary pointwise product is the pointwise product of third coordinates.
theorem function_quadruple_mul_second_second_first[T, A: Mul, B: Mul, C: Mul, D: Mul](
    p: Pair[T -> A, Pair[T -> B, Pair[T -> C, T -> D]]],
    q: Pair[T -> A, Pair[T -> B, Pair[T -> C, T -> D]]]
) {
    function_quadruple_mul(p, q).second.second.first =
        pointwise_mul(p.second.second.first, q.second.second.first)
} by {
    function_quadruple_mul(p, q) = Pair.new(pointwise_mul(p.first, q.first),
        Pair.new(pointwise_mul(p.second.first, q.second.first),
            Pair.new(pointwise_mul(p.second.second.first, q.second.second.first),
                pointwise_mul(p.second.second.second, q.second.second.second))))
    function_quadruple_mul(p, q).second =
        Pair.new(pointwise_mul(p.second.first, q.second.first),
            Pair.new(pointwise_mul(p.second.second.first, q.second.second.first),
                pointwise_mul(p.second.second.second, q.second.second.second)))
    function_quadruple_mul(p, q).second.second =
        Pair.new(pointwise_mul(p.second.second.first, q.second.second.first),
            pointwise_mul(p.second.second.second, q.second.second.second))
}

/// The fourth coordinate of a quaternary pointwise product is the pointwise product of fourth coordinates.
theorem function_quadruple_mul_second_second_second[T, A: Mul, B: Mul, C: Mul, D: Mul](
    p: Pair[T -> A, Pair[T -> B, Pair[T -> C, T -> D]]],
    q: Pair[T -> A, Pair[T -> B, Pair[T -> C, T -> D]]]
) {
    function_quadruple_mul(p, q).second.second.second =
        pointwise_mul(p.second.second.second, q.second.second.second)
} by {
    function_quadruple_mul(p, q) = Pair.new(pointwise_mul(p.first, q.first),
        Pair.new(pointwise_mul(p.second.first, q.second.first),
            Pair.new(pointwise_mul(p.second.second.first, q.second.second.first),
                pointwise_mul(p.second.second.second, q.second.second.second))))
    function_quadruple_mul(p, q).second =
        Pair.new(pointwise_mul(p.second.first, q.second.first),
            Pair.new(pointwise_mul(p.second.second.first, q.second.second.first),
                pointwise_mul(p.second.second.second, q.second.second.second)))
    function_quadruple_mul(p, q).second.second =
        Pair.new(pointwise_mul(p.second.second.first, q.second.second.first),
            pointwise_mul(p.second.second.second, q.second.second.second))
}

/// The first coordinate of the quaternary pointwise zero is the pointwise zero.
theorem function_quadruple_zero_first[T, A: Zero, B: Zero, C: Zero, D: Zero] {
    function_quadruple_zero[T, A, B, C, D].first = pointwise_zero[T, A]
}

/// The second coordinate of the quaternary pointwise zero is the pointwise zero.
theorem function_quadruple_zero_second_first[T, A: Zero, B: Zero, C: Zero, D: Zero] {
    function_quadruple_zero[T, A, B, C, D].second.first = pointwise_zero[T, B]
}

/// The third coordinate of the quaternary pointwise zero is the pointwise zero.
theorem function_quadruple_zero_second_second_first[T, A: Zero, B: Zero, C: Zero, D: Zero] {
    function_quadruple_zero[T, A, B, C, D].second.second.first = pointwise_zero[T, C]
}

/// The fourth coordinate of the quaternary pointwise zero is the pointwise zero.
theorem function_quadruple_zero_second_second_second[T, A: Zero, B: Zero, C: Zero, D: Zero] {
    function_quadruple_zero[T, A, B, C, D].second.second.second = pointwise_zero[T, D]
}

/// The first coordinate of the quaternary pointwise one is the pointwise one.
theorem function_quadruple_one_first[T, A: One, B: One, C: One, D: One] {
    function_quadruple_one[T, A, B, C, D].first = pointwise_one[T, A]
}

/// The second coordinate of the quaternary pointwise one is the pointwise one.
theorem function_quadruple_one_second_first[T, A: One, B: One, C: One, D: One] {
    function_quadruple_one[T, A, B, C, D].second.first = pointwise_one[T, B]
}

/// The third coordinate of the quaternary pointwise one is the pointwise one.
theorem function_quadruple_one_second_second_first[T, A: One, B: One, C: One, D: One] {
    function_quadruple_one[T, A, B, C, D].second.second.first = pointwise_one[T, C]
}

/// The fourth coordinate of the quaternary pointwise one is the pointwise one.
theorem function_quadruple_one_second_second_second[T, A: One, B: One, C: One, D: One] {
    function_quadruple_one[T, A, B, C, D].second.second.second = pointwise_one[T, D]
}

/// The first coordinate of quaternary pointwise negation is pointwise negation.
theorem function_quadruple_neg_first[T, A: Neg, B: Neg, C: Neg, D: Neg](
    p: Pair[T -> A, Pair[T -> B, Pair[T -> C, T -> D]]]
) {
    function_quadruple_neg(p).first = pointwise_neg(p.first)
}

/// The second coordinate of quaternary pointwise negation is pointwise negation.
theorem function_quadruple_neg_second_first[T, A: Neg, B: Neg, C: Neg, D: Neg](
    p: Pair[T -> A, Pair[T -> B, Pair[T -> C, T -> D]]]
) {
    function_quadruple_neg(p).second.first = pointwise_neg(p.second.first)
} by {
    function_quadruple_neg(p).second =
        Pair.new(pointwise_neg(p.second.first),
            Pair.new(pointwise_neg(p.second.second.first),
                pointwise_neg(p.second.second.second)))
}

/// The third coordinate of quaternary pointwise negation is pointwise negation.
theorem function_quadruple_neg_second_second_first[T, A: Neg, B: Neg, C: Neg, D: Neg](
    p: Pair[T -> A, Pair[T -> B, Pair[T -> C, T -> D]]]
) {
    function_quadruple_neg(p).second.second.first = pointwise_neg(p.second.second.first)
} by {
    function_quadruple_neg(p).second =
        Pair.new(pointwise_neg(p.second.first),
            Pair.new(pointwise_neg(p.second.second.first),
                pointwise_neg(p.second.second.second)))
    function_quadruple_neg(p).second.second =
        Pair.new(pointwise_neg(p.second.second.first),
            pointwise_neg(p.second.second.second))
}

/// The fourth coordinate of quaternary pointwise negation is pointwise negation.
theorem function_quadruple_neg_second_second_second[T, A: Neg, B: Neg, C: Neg, D: Neg](
    p: Pair[T -> A, Pair[T -> B, Pair[T -> C, T -> D]]]
) {
    function_quadruple_neg(p).second.second.second = pointwise_neg(p.second.second.second)
} by {
    function_quadruple_neg(p).second =
        Pair.new(pointwise_neg(p.second.first),
            Pair.new(pointwise_neg(p.second.second.first),
                pointwise_neg(p.second.second.second)))
    function_quadruple_neg(p).second.second =
        Pair.new(pointwise_neg(p.second.second.first),
            pointwise_neg(p.second.second.second))
}

/// The first coordinate of quaternary pointwise inversion is pointwise inversion.
theorem function_quadruple_inverse_first[T, A: Group, B: Group, C: Group, D: Group](
    p: Pair[T -> A, Pair[T -> B, Pair[T -> C, T -> D]]]
) {
    function_quadruple_inverse(p).first = pointwise_inverse(p.first)
}

/// The second coordinate of quaternary pointwise inversion is pointwise inversion.
theorem function_quadruple_inverse_second_first[T, A: Group, B: Group, C: Group, D: Group](
    p: Pair[T -> A, Pair[T -> B, Pair[T -> C, T -> D]]]
) {
    function_quadruple_inverse(p).second.first = pointwise_inverse(p.second.first)
} by {
    function_quadruple_inverse(p).second =
        Pair.new(pointwise_inverse(p.second.first),
            Pair.new(pointwise_inverse(p.second.second.first),
                pointwise_inverse(p.second.second.second)))
}

/// The third coordinate of quaternary pointwise inversion is pointwise inversion.
theorem function_quadruple_inverse_second_second_first[T, A: Group, B: Group, C: Group, D: Group](
    p: Pair[T -> A, Pair[T -> B, Pair[T -> C, T -> D]]]
) {
    function_quadruple_inverse(p).second.second.first =
        pointwise_inverse(p.second.second.first)
} by {
    function_quadruple_inverse(p).second =
        Pair.new(pointwise_inverse(p.second.first),
            Pair.new(pointwise_inverse(p.second.second.first),
                pointwise_inverse(p.second.second.second)))
    function_quadruple_inverse(p).second.second =
        Pair.new(pointwise_inverse(p.second.second.first),
            pointwise_inverse(p.second.second.second))
}

/// The fourth coordinate of quaternary pointwise inversion is pointwise inversion.
theorem function_quadruple_inverse_second_second_second[T, A: Group, B: Group, C: Group, D: Group](
    p: Pair[T -> A, Pair[T -> B, Pair[T -> C, T -> D]]]
) {
    function_quadruple_inverse(p).second.second.second =
        pointwise_inverse(p.second.second.second)
} by {
    function_quadruple_inverse(p).second =
        Pair.new(pointwise_inverse(p.second.first),
            Pair.new(pointwise_inverse(p.second.second.first),
                pointwise_inverse(p.second.second.second)))
    function_quadruple_inverse(p).second.second =
        Pair.new(pointwise_inverse(p.second.second.first),
            pointwise_inverse(p.second.second.second))
}


/// Pointwise addition is associative on a quaternary product of function spaces.
theorem function_quadruple_add_assoc[T, A: AddSemigroup, B: AddSemigroup, C: AddSemigroup, D: AddSemigroup](
    p: Pair[T -> A, Pair[T -> B, Pair[T -> C, T -> D]]],
    q: Pair[T -> A, Pair[T -> B, Pair[T -> C, T -> D]]],
    r: Pair[T -> A, Pair[T -> B, Pair[T -> C, T -> D]]]
) {
    function_quadruple_add(p, function_quadruple_add(q, r)) =
    function_quadruple_add(function_quadruple_add(p, q), r)
} by {
    let lhs = function_quadruple_add(p, function_quadruple_add(q, r))
    let rhs = function_quadruple_add(function_quadruple_add(p, q), r)
    lhs.first = pointwise_add(p.first, function_quadruple_add(q, r).first)
    function_quadruple_add(q, r).first = pointwise_add(q.first, r.first)
    lhs.first = pointwise_add(p.first, pointwise_add(q.first, r.first))
    rhs.first = pointwise_add(function_quadruple_add(p, q).first, r.first)
    function_quadruple_add(p, q).first = pointwise_add(p.first, q.first)
    rhs.first = pointwise_add(pointwise_add(p.first, q.first), r.first)
    pointwise_add_assoc(p.first, q.first, r.first)
    lhs.first = rhs.first
    lhs.second.first = pointwise_add(p.second.first, function_quadruple_add(q, r).second.first)
    function_quadruple_add(q, r).second.first = pointwise_add(q.second.first, r.second.first)
    lhs.second.first = pointwise_add(p.second.first, pointwise_add(q.second.first, r.second.first))
    rhs.second.first = pointwise_add(function_quadruple_add(p, q).second.first, r.second.first)
    function_quadruple_add(p, q).second.first = pointwise_add(p.second.first, q.second.first)
    rhs.second.first = pointwise_add(pointwise_add(p.second.first, q.second.first), r.second.first)
    pointwise_add_assoc(p.second.first, q.second.first, r.second.first)
    lhs.second.first = rhs.second.first
    lhs.second.second.first = pointwise_add(p.second.second.first,
        function_quadruple_add(q, r).second.second.first)
    function_quadruple_add(q, r).second.second.first =
        pointwise_add(q.second.second.first, r.second.second.first)
    lhs.second.second.first = pointwise_add(p.second.second.first,
        pointwise_add(q.second.second.first, r.second.second.first))
    rhs.second.second.first = pointwise_add(
        function_quadruple_add(p, q).second.second.first, r.second.second.first)
    function_quadruple_add(p, q).second.second.first =
        pointwise_add(p.second.second.first, q.second.second.first)
    rhs.second.second.first = pointwise_add(
        pointwise_add(p.second.second.first, q.second.second.first), r.second.second.first)
    pointwise_add_assoc(p.second.second.first, q.second.second.first, r.second.second.first)
    lhs.second.second.first = rhs.second.second.first
    lhs.second.second.second = pointwise_add(p.second.second.second,
        function_quadruple_add(q, r).second.second.second)
    function_quadruple_add(q, r).second.second.second =
        pointwise_add(q.second.second.second, r.second.second.second)
    lhs.second.second.second = pointwise_add(p.second.second.second,
        pointwise_add(q.second.second.second, r.second.second.second))
    rhs.second.second.second = pointwise_add(
        function_quadruple_add(p, q).second.second.second, r.second.second.second)
    function_quadruple_add(p, q).second.second.second =
        pointwise_add(p.second.second.second, q.second.second.second)
    rhs.second.second.second = pointwise_add(
        pointwise_add(p.second.second.second, q.second.second.second), r.second.second.second)
    pointwise_add_assoc(p.second.second.second, q.second.second.second, r.second.second.second)
    lhs.second.second.second = rhs.second.second.second
    function_quadruple_ext(lhs, rhs)
}

/// Pointwise addition is commutative on a quaternary product of function spaces.
theorem function_quadruple_add_comm[T, A: AddCommSemigroup, B: AddCommSemigroup, C: AddCommSemigroup, D: AddCommSemigroup](
    p: Pair[T -> A, Pair[T -> B, Pair[T -> C, T -> D]]],
    q: Pair[T -> A, Pair[T -> B, Pair[T -> C, T -> D]]]
) {
    function_quadruple_add(p, q) = function_quadruple_add(q, p)
} by {
    let lhs = function_quadruple_add(p, q)
    let rhs = function_quadruple_add(q, p)
    lhs.first = pointwise_add(p.first, q.first)
    rhs.first = pointwise_add(q.first, p.first)
    pointwise_add_comm(p.first, q.first)
    lhs.first = rhs.first
    lhs.second.first = pointwise_add(p.second.first, q.second.first)
    rhs.second.first = pointwise_add(q.second.first, p.second.first)
    pointwise_add_comm(p.second.first, q.second.first)
    lhs.second.first = rhs.second.first
    lhs.second.second.first = pointwise_add(p.second.second.first, q.second.second.first)
    rhs.second.second.first = pointwise_add(q.second.second.first, p.second.second.first)
    pointwise_add_comm(p.second.second.first, q.second.second.first)
    lhs.second.second.first = rhs.second.second.first
    lhs.second.second.second = pointwise_add(p.second.second.second, q.second.second.second)
    rhs.second.second.second = pointwise_add(q.second.second.second, p.second.second.second)
    pointwise_add_comm(p.second.second.second, q.second.second.second)
    lhs.second.second.second = rhs.second.second.second
    function_quadruple_ext(lhs, rhs)
}

/// Adding the quaternary pointwise zero on the right changes no coordinate.
theorem function_quadruple_add_zero_right[T, A: AddMonoid, B: AddMonoid, C: AddMonoid, D: AddMonoid](
    p: Pair[T -> A, Pair[T -> B, Pair[T -> C, T -> D]]]
) {
    function_quadruple_add(p, function_quadruple_zero[T, A, B, C, D]) = p
} by {
    let lhs = function_quadruple_add(p, function_quadruple_zero[T, A, B, C, D])
    lhs.first = pointwise_add(p.first, function_quadruple_zero[T, A, B, C, D].first)
    function_quadruple_zero[T, A, B, C, D].first = pointwise_zero[T, A]
    lhs.first = pointwise_add(p.first, pointwise_zero[T, A])
    pointwise_add_zero_right(p.first)
    lhs.first = p.first
    lhs.second.first = pointwise_add(p.second.first,
        function_quadruple_zero[T, A, B, C, D].second.first)
    function_quadruple_zero[T, A, B, C, D].second.first = pointwise_zero[T, B]
    lhs.second.first = pointwise_add(p.second.first, pointwise_zero[T, B])
    pointwise_add_zero_right(p.second.first)
    lhs.second.first = p.second.first
    lhs.second.second.first = pointwise_add(p.second.second.first,
        function_quadruple_zero[T, A, B, C, D].second.second.first)
    function_quadruple_zero[T, A, B, C, D].second.second.first = pointwise_zero[T, C]
    lhs.second.second.first = pointwise_add(p.second.second.first, pointwise_zero[T, C])
    pointwise_add_zero_right(p.second.second.first)
    lhs.second.second.first = p.second.second.first
    lhs.second.second.second = pointwise_add(p.second.second.second,
        function_quadruple_zero[T, A, B, C, D].second.second.second)
    function_quadruple_zero[T, A, B, C, D].second.second.second = pointwise_zero[T, D]
    lhs.second.second.second = pointwise_add(p.second.second.second, pointwise_zero[T, D])
    pointwise_add_zero_right(p.second.second.second)
    lhs.second.second.second = p.second.second.second
    function_quadruple_ext(lhs, p)
}

/// Adding the quaternary pointwise zero on the left changes no coordinate.
theorem function_quadruple_add_zero_left[T, A: AddMonoid, B: AddMonoid, C: AddMonoid, D: AddMonoid](
    p: Pair[T -> A, Pair[T -> B, Pair[T -> C, T -> D]]]
) {
    function_quadruple_add(function_quadruple_zero[T, A, B, C, D], p) = p
} by {
    let lhs = function_quadruple_add(function_quadruple_zero[T, A, B, C, D], p)
    lhs.first = pointwise_add(function_quadruple_zero[T, A, B, C, D].first, p.first)
    function_quadruple_zero[T, A, B, C, D].first = pointwise_zero[T, A]
    lhs.first = pointwise_add(pointwise_zero[T, A], p.first)
    pointwise_add_zero_left(p.first)
    lhs.first = p.first
    lhs.second.first = pointwise_add(
        function_quadruple_zero[T, A, B, C, D].second.first, p.second.first)
    function_quadruple_zero[T, A, B, C, D].second.first = pointwise_zero[T, B]
    lhs.second.first = pointwise_add(pointwise_zero[T, B], p.second.first)
    pointwise_add_zero_left(p.second.first)
    lhs.second.first = p.second.first
    lhs.second.second.first = pointwise_add(
        function_quadruple_zero[T, A, B, C, D].second.second.first, p.second.second.first)
    function_quadruple_zero[T, A, B, C, D].second.second.first = pointwise_zero[T, C]
    lhs.second.second.first = pointwise_add(pointwise_zero[T, C], p.second.second.first)
    pointwise_add_zero_left(p.second.second.first)
    lhs.second.second.first = p.second.second.first
    lhs.second.second.second = pointwise_add(
        function_quadruple_zero[T, A, B, C, D].second.second.second, p.second.second.second)
    function_quadruple_zero[T, A, B, C, D].second.second.second = pointwise_zero[T, D]
    lhs.second.second.second = pointwise_add(pointwise_zero[T, D], p.second.second.second)
    pointwise_add_zero_left(p.second.second.second)
    lhs.second.second.second = p.second.second.second
    function_quadruple_ext(lhs, p)
}

/// Pointwise negation is a right additive inverse on a quaternary product of function spaces.
theorem function_quadruple_add_neg_right[T, A: AddGroup, B: AddGroup, C: AddGroup, D: AddGroup](
    p: Pair[T -> A, Pair[T -> B, Pair[T -> C, T -> D]]]
) {
    function_quadruple_add(p, function_quadruple_neg(p)) =
    function_quadruple_zero[T, A, B, C, D]
} by {
    let lhs = function_quadruple_add(p, function_quadruple_neg(p))
    lhs.first = pointwise_add(p.first, function_quadruple_neg(p).first)
    function_quadruple_neg(p).first = pointwise_neg(p.first)
    lhs.first = pointwise_add(p.first, pointwise_neg(p.first))
    pointwise_add_neg_right(p.first)
    lhs.first = pointwise_zero[T, A]
    function_quadruple_zero[T, A, B, C, D].first = pointwise_zero[T, A]
    lhs.second.first = pointwise_add(p.second.first, function_quadruple_neg(p).second.first)
    function_quadruple_neg(p).second.first = pointwise_neg(p.second.first)
    lhs.second.first = pointwise_add(p.second.first, pointwise_neg(p.second.first))
    pointwise_add_neg_right(p.second.first)
    lhs.second.first = pointwise_zero[T, B]
    function_quadruple_zero[T, A, B, C, D].second.first = pointwise_zero[T, B]
    lhs.second.second.first = pointwise_add(p.second.second.first,
        function_quadruple_neg(p).second.second.first)
    function_quadruple_neg(p).second.second.first = pointwise_neg(p.second.second.first)
    lhs.second.second.first = pointwise_add(p.second.second.first,
        pointwise_neg(p.second.second.first))
    pointwise_add_neg_right(p.second.second.first)
    lhs.second.second.first = pointwise_zero[T, C]
    function_quadruple_zero[T, A, B, C, D].second.second.first = pointwise_zero[T, C]
    lhs.second.second.second = pointwise_add(p.second.second.second,
        function_quadruple_neg(p).second.second.second)
    function_quadruple_neg(p).second.second.second = pointwise_neg(p.second.second.second)
    lhs.second.second.second = pointwise_add(p.second.second.second,
        pointwise_neg(p.second.second.second))
    pointwise_add_neg_right(p.second.second.second)
    lhs.second.second.second = pointwise_zero[T, D]
    function_quadruple_zero[T, A, B, C, D].second.second.second = pointwise_zero[T, D]
    function_quadruple_ext(lhs, function_quadruple_zero[T, A, B, C, D])
}

/// Pointwise negation is a left additive inverse on a quaternary product of function spaces.
theorem function_quadruple_add_neg_left[T, A: AddGroup, B: AddGroup, C: AddGroup, D: AddGroup](
    p: Pair[T -> A, Pair[T -> B, Pair[T -> C, T -> D]]]
) {
    function_quadruple_add(function_quadruple_neg(p), p) =
    function_quadruple_zero[T, A, B, C, D]
} by {
    let lhs = function_quadruple_add(function_quadruple_neg(p), p)
    lhs.first = pointwise_add(function_quadruple_neg(p).first, p.first)
    function_quadruple_neg(p).first = pointwise_neg(p.first)
    lhs.first = pointwise_add(pointwise_neg(p.first), p.first)
    pointwise_add_neg_left(p.first)
    lhs.first = pointwise_zero[T, A]
    function_quadruple_zero[T, A, B, C, D].first = pointwise_zero[T, A]
    lhs.second.first = pointwise_add(function_quadruple_neg(p).second.first, p.second.first)
    function_quadruple_neg(p).second.first = pointwise_neg(p.second.first)
    lhs.second.first = pointwise_add(pointwise_neg(p.second.first), p.second.first)
    pointwise_add_neg_left(p.second.first)
    lhs.second.first = pointwise_zero[T, B]
    function_quadruple_zero[T, A, B, C, D].second.first = pointwise_zero[T, B]
    lhs.second.second.first = pointwise_add(function_quadruple_neg(p).second.second.first,
        p.second.second.first)
    function_quadruple_neg(p).second.second.first = pointwise_neg(p.second.second.first)
    lhs.second.second.first = pointwise_add(pointwise_neg(p.second.second.first),
        p.second.second.first)
    pointwise_add_neg_left(p.second.second.first)
    lhs.second.second.first = pointwise_zero[T, C]
    function_quadruple_zero[T, A, B, C, D].second.second.first = pointwise_zero[T, C]
    lhs.second.second.second = pointwise_add(function_quadruple_neg(p).second.second.second,
        p.second.second.second)
    function_quadruple_neg(p).second.second.second = pointwise_neg(p.second.second.second)
    lhs.second.second.second = pointwise_add(pointwise_neg(p.second.second.second),
        p.second.second.second)
    pointwise_add_neg_left(p.second.second.second)
    lhs.second.second.second = pointwise_zero[T, D]
    function_quadruple_zero[T, A, B, C, D].second.second.second = pointwise_zero[T, D]
    function_quadruple_ext(lhs, function_quadruple_zero[T, A, B, C, D])
}

/// Pointwise multiplication is associative on a quaternary product of function spaces.
theorem function_quadruple_mul_assoc[T, A: Semigroup, B: Semigroup, C: Semigroup, D: Semigroup](
    p: Pair[T -> A, Pair[T -> B, Pair[T -> C, T -> D]]],
    q: Pair[T -> A, Pair[T -> B, Pair[T -> C, T -> D]]],
    r: Pair[T -> A, Pair[T -> B, Pair[T -> C, T -> D]]]
) {
    function_quadruple_mul(p, function_quadruple_mul(q, r)) =
    function_quadruple_mul(function_quadruple_mul(p, q), r)
} by {
    let lhs = function_quadruple_mul(p, function_quadruple_mul(q, r))
    let rhs = function_quadruple_mul(function_quadruple_mul(p, q), r)
    lhs.first = pointwise_mul(p.first, function_quadruple_mul(q, r).first)
    function_quadruple_mul(q, r).first = pointwise_mul(q.first, r.first)
    lhs.first = pointwise_mul(p.first, pointwise_mul(q.first, r.first))
    rhs.first = pointwise_mul(function_quadruple_mul(p, q).first, r.first)
    function_quadruple_mul(p, q).first = pointwise_mul(p.first, q.first)
    rhs.first = pointwise_mul(pointwise_mul(p.first, q.first), r.first)
    pointwise_mul_assoc(p.first, q.first, r.first)
    lhs.first = rhs.first
    lhs.second.first = pointwise_mul(p.second.first, function_quadruple_mul(q, r).second.first)
    function_quadruple_mul(q, r).second.first = pointwise_mul(q.second.first, r.second.first)
    lhs.second.first = pointwise_mul(p.second.first, pointwise_mul(q.second.first, r.second.first))
    rhs.second.first = pointwise_mul(function_quadruple_mul(p, q).second.first, r.second.first)
    function_quadruple_mul(p, q).second.first = pointwise_mul(p.second.first, q.second.first)
    rhs.second.first = pointwise_mul(pointwise_mul(p.second.first, q.second.first), r.second.first)
    pointwise_mul_assoc(p.second.first, q.second.first, r.second.first)
    lhs.second.first = rhs.second.first
    lhs.second.second.first = pointwise_mul(p.second.second.first,
        function_quadruple_mul(q, r).second.second.first)
    function_quadruple_mul(q, r).second.second.first =
        pointwise_mul(q.second.second.first, r.second.second.first)
    lhs.second.second.first = pointwise_mul(p.second.second.first,
        pointwise_mul(q.second.second.first, r.second.second.first))
    rhs.second.second.first = pointwise_mul(
        function_quadruple_mul(p, q).second.second.first, r.second.second.first)
    function_quadruple_mul(p, q).second.second.first =
        pointwise_mul(p.second.second.first, q.second.second.first)
    rhs.second.second.first = pointwise_mul(
        pointwise_mul(p.second.second.first, q.second.second.first), r.second.second.first)
    pointwise_mul_assoc(p.second.second.first, q.second.second.first, r.second.second.first)
    lhs.second.second.first = rhs.second.second.first
    lhs.second.second.second = pointwise_mul(p.second.second.second,
        function_quadruple_mul(q, r).second.second.second)
    function_quadruple_mul(q, r).second.second.second =
        pointwise_mul(q.second.second.second, r.second.second.second)
    lhs.second.second.second = pointwise_mul(p.second.second.second,
        pointwise_mul(q.second.second.second, r.second.second.second))
    rhs.second.second.second = pointwise_mul(
        function_quadruple_mul(p, q).second.second.second, r.second.second.second)
    function_quadruple_mul(p, q).second.second.second =
        pointwise_mul(p.second.second.second, q.second.second.second)
    rhs.second.second.second = pointwise_mul(
        pointwise_mul(p.second.second.second, q.second.second.second), r.second.second.second)
    pointwise_mul_assoc(p.second.second.second, q.second.second.second, r.second.second.second)
    lhs.second.second.second = rhs.second.second.second
    function_quadruple_ext(lhs, rhs)
}

/// Pointwise multiplication is commutative on a quaternary product of function spaces.
theorem function_quadruple_mul_comm[T, A: CommSemigroup, B: CommSemigroup, C: CommSemigroup, D: CommSemigroup](
    p: Pair[T -> A, Pair[T -> B, Pair[T -> C, T -> D]]],
    q: Pair[T -> A, Pair[T -> B, Pair[T -> C, T -> D]]]
) {
    function_quadruple_mul(p, q) = function_quadruple_mul(q, p)
} by {
    let lhs = function_quadruple_mul(p, q)
    let rhs = function_quadruple_mul(q, p)
    lhs.first = pointwise_mul(p.first, q.first)
    rhs.first = pointwise_mul(q.first, p.first)
    pointwise_mul_comm(p.first, q.first)
    lhs.first = rhs.first
    lhs.second.first = pointwise_mul(p.second.first, q.second.first)
    rhs.second.first = pointwise_mul(q.second.first, p.second.first)
    pointwise_mul_comm(p.second.first, q.second.first)
    lhs.second.first = rhs.second.first
    lhs.second.second.first = pointwise_mul(p.second.second.first, q.second.second.first)
    rhs.second.second.first = pointwise_mul(q.second.second.first, p.second.second.first)
    pointwise_mul_comm(p.second.second.first, q.second.second.first)
    lhs.second.second.first = rhs.second.second.first
    lhs.second.second.second = pointwise_mul(p.second.second.second, q.second.second.second)
    rhs.second.second.second = pointwise_mul(q.second.second.second, p.second.second.second)
    pointwise_mul_comm(p.second.second.second, q.second.second.second)
    lhs.second.second.second = rhs.second.second.second
    function_quadruple_ext(lhs, rhs)
}

/// Multiplying by the quaternary pointwise one on the right changes no coordinate.
theorem function_quadruple_mul_one_right[T, A: Monoid, B: Monoid, C: Monoid, D: Monoid](
    p: Pair[T -> A, Pair[T -> B, Pair[T -> C, T -> D]]]
) {
    function_quadruple_mul(p, function_quadruple_one[T, A, B, C, D]) = p
} by {
    let lhs = function_quadruple_mul(p, function_quadruple_one[T, A, B, C, D])
    lhs.first = pointwise_mul(p.first, function_quadruple_one[T, A, B, C, D].first)
    function_quadruple_one[T, A, B, C, D].first = pointwise_one[T, A]
    lhs.first = pointwise_mul(p.first, pointwise_one[T, A])
    pointwise_mul_one_right(p.first)
    lhs.first = p.first
    lhs.second.first = pointwise_mul(p.second.first,
        function_quadruple_one[T, A, B, C, D].second.first)
    function_quadruple_one[T, A, B, C, D].second.first = pointwise_one[T, B]
    lhs.second.first = pointwise_mul(p.second.first, pointwise_one[T, B])
    pointwise_mul_one_right(p.second.first)
    lhs.second.first = p.second.first
    lhs.second.second.first = pointwise_mul(p.second.second.first,
        function_quadruple_one[T, A, B, C, D].second.second.first)
    function_quadruple_one[T, A, B, C, D].second.second.first = pointwise_one[T, C]
    lhs.second.second.first = pointwise_mul(p.second.second.first, pointwise_one[T, C])
    pointwise_mul_one_right(p.second.second.first)
    lhs.second.second.first = p.second.second.first
    lhs.second.second.second = pointwise_mul(p.second.second.second,
        function_quadruple_one[T, A, B, C, D].second.second.second)
    function_quadruple_one[T, A, B, C, D].second.second.second = pointwise_one[T, D]
    lhs.second.second.second = pointwise_mul(p.second.second.second, pointwise_one[T, D])
    pointwise_mul_one_right(p.second.second.second)
    lhs.second.second.second = p.second.second.second
    function_quadruple_ext(lhs, p)
}

/// Multiplying by the quaternary pointwise one on the left changes no coordinate.
theorem function_quadruple_mul_one_left[T, A: Monoid, B: Monoid, C: Monoid, D: Monoid](
    p: Pair[T -> A, Pair[T -> B, Pair[T -> C, T -> D]]]
) {
    function_quadruple_mul(function_quadruple_one[T, A, B, C, D], p) = p
} by {
    let lhs = function_quadruple_mul(function_quadruple_one[T, A, B, C, D], p)
    lhs.first = pointwise_mul(function_quadruple_one[T, A, B, C, D].first, p.first)
    function_quadruple_one[T, A, B, C, D].first = pointwise_one[T, A]
    lhs.first = pointwise_mul(pointwise_one[T, A], p.first)
    pointwise_mul_one_left(p.first)
    lhs.first = p.first
    lhs.second.first = pointwise_mul(
        function_quadruple_one[T, A, B, C, D].second.first, p.second.first)
    function_quadruple_one[T, A, B, C, D].second.first = pointwise_one[T, B]
    lhs.second.first = pointwise_mul(pointwise_one[T, B], p.second.first)
    pointwise_mul_one_left(p.second.first)
    lhs.second.first = p.second.first
    lhs.second.second.first = pointwise_mul(
        function_quadruple_one[T, A, B, C, D].second.second.first, p.second.second.first)
    function_quadruple_one[T, A, B, C, D].second.second.first = pointwise_one[T, C]
    lhs.second.second.first = pointwise_mul(pointwise_one[T, C], p.second.second.first)
    pointwise_mul_one_left(p.second.second.first)
    lhs.second.second.first = p.second.second.first
    lhs.second.second.second = pointwise_mul(
        function_quadruple_one[T, A, B, C, D].second.second.second, p.second.second.second)
    function_quadruple_one[T, A, B, C, D].second.second.second = pointwise_one[T, D]
    lhs.second.second.second = pointwise_mul(pointwise_one[T, D], p.second.second.second)
    pointwise_mul_one_left(p.second.second.second)
    lhs.second.second.second = p.second.second.second
    function_quadruple_ext(lhs, p)
}

/// Pointwise inversion is a right inverse on a quaternary product of function spaces.
theorem function_quadruple_mul_inverse_right[T, A: Group, B: Group, C: Group, D: Group](
    p: Pair[T -> A, Pair[T -> B, Pair[T -> C, T -> D]]]
) {
    function_quadruple_mul(p, function_quadruple_inverse(p)) =
    function_quadruple_one[T, A, B, C, D]
} by {
    let lhs = function_quadruple_mul(p, function_quadruple_inverse(p))
    lhs.first = pointwise_mul(p.first, function_quadruple_inverse(p).first)
    function_quadruple_inverse(p).first = pointwise_inverse(p.first)
    lhs.first = pointwise_mul(p.first, pointwise_inverse(p.first))
    pointwise_mul_inverse_right(p.first)
    lhs.first = pointwise_one[T, A]
    function_quadruple_one[T, A, B, C, D].first = pointwise_one[T, A]
    lhs.second.first = pointwise_mul(p.second.first, function_quadruple_inverse(p).second.first)
    function_quadruple_inverse(p).second.first = pointwise_inverse(p.second.first)
    lhs.second.first = pointwise_mul(p.second.first, pointwise_inverse(p.second.first))
    pointwise_mul_inverse_right(p.second.first)
    lhs.second.first = pointwise_one[T, B]
    function_quadruple_one[T, A, B, C, D].second.first = pointwise_one[T, B]
    lhs.second.second.first = pointwise_mul(p.second.second.first,
        function_quadruple_inverse(p).second.second.first)
    function_quadruple_inverse(p).second.second.first = pointwise_inverse(p.second.second.first)
    lhs.second.second.first = pointwise_mul(p.second.second.first,
        pointwise_inverse(p.second.second.first))
    pointwise_mul_inverse_right(p.second.second.first)
    lhs.second.second.first = pointwise_one[T, C]
    function_quadruple_one[T, A, B, C, D].second.second.first = pointwise_one[T, C]
    lhs.second.second.second = pointwise_mul(p.second.second.second,
        function_quadruple_inverse(p).second.second.second)
    function_quadruple_inverse(p).second.second.second = pointwise_inverse(p.second.second.second)
    lhs.second.second.second = pointwise_mul(p.second.second.second,
        pointwise_inverse(p.second.second.second))
    pointwise_mul_inverse_right(p.second.second.second)
    lhs.second.second.second = pointwise_one[T, D]
    function_quadruple_one[T, A, B, C, D].second.second.second = pointwise_one[T, D]
    function_quadruple_ext(lhs, function_quadruple_one[T, A, B, C, D])
}

/// Pointwise inversion is a left inverse on a quaternary product of function spaces.
theorem function_quadruple_mul_inverse_left[T, A: Group, B: Group, C: Group, D: Group](
    p: Pair[T -> A, Pair[T -> B, Pair[T -> C, T -> D]]]
) {
    function_quadruple_mul(function_quadruple_inverse(p), p) =
    function_quadruple_one[T, A, B, C, D]
} by {
    let lhs = function_quadruple_mul(function_quadruple_inverse(p), p)
    lhs.first = pointwise_mul(function_quadruple_inverse(p).first, p.first)
    function_quadruple_inverse(p).first = pointwise_inverse(p.first)
    lhs.first = pointwise_mul(pointwise_inverse(p.first), p.first)
    pointwise_mul_inverse_left(p.first)
    lhs.first = pointwise_one[T, A]
    function_quadruple_one[T, A, B, C, D].first = pointwise_one[T, A]
    lhs.second.first = pointwise_mul(function_quadruple_inverse(p).second.first, p.second.first)
    function_quadruple_inverse(p).second.first = pointwise_inverse(p.second.first)
    lhs.second.first = pointwise_mul(pointwise_inverse(p.second.first), p.second.first)
    pointwise_mul_inverse_left(p.second.first)
    lhs.second.first = pointwise_one[T, B]
    function_quadruple_one[T, A, B, C, D].second.first = pointwise_one[T, B]
    lhs.second.second.first = pointwise_mul(function_quadruple_inverse(p).second.second.first,
        p.second.second.first)
    function_quadruple_inverse(p).second.second.first = pointwise_inverse(p.second.second.first)
    lhs.second.second.first = pointwise_mul(pointwise_inverse(p.second.second.first),
        p.second.second.first)
    pointwise_mul_inverse_left(p.second.second.first)
    lhs.second.second.first = pointwise_one[T, C]
    function_quadruple_one[T, A, B, C, D].second.second.first = pointwise_one[T, C]
    lhs.second.second.second = pointwise_mul(function_quadruple_inverse(p).second.second.second,
        p.second.second.second)
    function_quadruple_inverse(p).second.second.second = pointwise_inverse(p.second.second.second)
    lhs.second.second.second = pointwise_mul(pointwise_inverse(p.second.second.second),
        p.second.second.second)
    pointwise_mul_inverse_left(p.second.second.second)
    lhs.second.second.second = pointwise_one[T, D]
    function_quadruple_one[T, A, B, C, D].second.second.second = pointwise_one[T, D]
    function_quadruple_ext(lhs, function_quadruple_one[T, A, B, C, D])
}

/// Pointwise multiplication distributes over quaternary pointwise addition on the left.
theorem function_quadruple_mul_distrib_left[T, A: Semiring, B: Semiring, C: Semiring, D: Semiring](
    p: Pair[T -> A, Pair[T -> B, Pair[T -> C, T -> D]]],
    q: Pair[T -> A, Pair[T -> B, Pair[T -> C, T -> D]]],
    r: Pair[T -> A, Pair[T -> B, Pair[T -> C, T -> D]]]
) {
    function_quadruple_mul(p, function_quadruple_add(q, r)) =
    function_quadruple_add(function_quadruple_mul(p, q), function_quadruple_mul(p, r))
} by {
    let lhs = function_quadruple_mul(p, function_quadruple_add(q, r))
    let rhs = function_quadruple_add(function_quadruple_mul(p, q), function_quadruple_mul(p, r))
    lhs.first = pointwise_mul(p.first, function_quadruple_add(q, r).first)
    function_quadruple_add(q, r).first = pointwise_add(q.first, r.first)
    lhs.first = pointwise_mul(p.first, pointwise_add(q.first, r.first))
    rhs.first = pointwise_add(function_quadruple_mul(p, q).first, function_quadruple_mul(p, r).first)
    function_quadruple_mul(p, q).first = pointwise_mul(p.first, q.first)
    function_quadruple_mul(p, r).first = pointwise_mul(p.first, r.first)
    rhs.first = pointwise_add(pointwise_mul(p.first, q.first), pointwise_mul(p.first, r.first))
    pointwise_mul_distrib_left(p.first, q.first, r.first)
    lhs.first = rhs.first
    lhs.second.first = pointwise_mul(p.second.first, function_quadruple_add(q, r).second.first)
    function_quadruple_add(q, r).second.first = pointwise_add(q.second.first, r.second.first)
    lhs.second.first = pointwise_mul(p.second.first, pointwise_add(q.second.first, r.second.first))
    rhs.second.first = pointwise_add(
        function_quadruple_mul(p, q).second.first,
        function_quadruple_mul(p, r).second.first)
    function_quadruple_mul(p, q).second.first = pointwise_mul(p.second.first, q.second.first)
    function_quadruple_mul(p, r).second.first = pointwise_mul(p.second.first, r.second.first)
    rhs.second.first =
        pointwise_add(pointwise_mul(p.second.first, q.second.first),
            pointwise_mul(p.second.first, r.second.first))
    pointwise_mul_distrib_left(p.second.first, q.second.first, r.second.first)
    lhs.second.first = rhs.second.first
    lhs.second.second.first = pointwise_mul(p.second.second.first,
        function_quadruple_add(q, r).second.second.first)
    function_quadruple_add(q, r).second.second.first =
        pointwise_add(q.second.second.first, r.second.second.first)
    lhs.second.second.first = pointwise_mul(p.second.second.first,
        pointwise_add(q.second.second.first, r.second.second.first))
    rhs.second.second.first = pointwise_add(
        function_quadruple_mul(p, q).second.second.first,
        function_quadruple_mul(p, r).second.second.first)
    function_quadruple_mul(p, q).second.second.first =
        pointwise_mul(p.second.second.first, q.second.second.first)
    function_quadruple_mul(p, r).second.second.first =
        pointwise_mul(p.second.second.first, r.second.second.first)
    rhs.second.second.first =
        pointwise_add(pointwise_mul(p.second.second.first, q.second.second.first),
            pointwise_mul(p.second.second.first, r.second.second.first))
    pointwise_mul_distrib_left(p.second.second.first, q.second.second.first, r.second.second.first)
    lhs.second.second.first = rhs.second.second.first
    lhs.second.second.second = pointwise_mul(p.second.second.second,
        function_quadruple_add(q, r).second.second.second)
    function_quadruple_add(q, r).second.second.second =
        pointwise_add(q.second.second.second, r.second.second.second)
    lhs.second.second.second = pointwise_mul(p.second.second.second,
        pointwise_add(q.second.second.second, r.second.second.second))
    rhs.second.second.second = pointwise_add(
        function_quadruple_mul(p, q).second.second.second,
        function_quadruple_mul(p, r).second.second.second)
    function_quadruple_mul(p, q).second.second.second =
        pointwise_mul(p.second.second.second, q.second.second.second)
    function_quadruple_mul(p, r).second.second.second =
        pointwise_mul(p.second.second.second, r.second.second.second)
    rhs.second.second.second =
        pointwise_add(pointwise_mul(p.second.second.second, q.second.second.second),
            pointwise_mul(p.second.second.second, r.second.second.second))
    pointwise_mul_distrib_left(p.second.second.second, q.second.second.second, r.second.second.second)
    lhs.second.second.second =
        pointwise_add(pointwise_mul(p.second.second.second, q.second.second.second),
            pointwise_mul(p.second.second.second, r.second.second.second))
    lhs.second.second.second = rhs.second.second.second
    function_quadruple_ext(lhs, rhs)
}

/// Pointwise multiplication distributes over quaternary pointwise addition on the right.
theorem function_quadruple_mul_distrib_right[T, A: Semiring, B: Semiring, C: Semiring, D: Semiring](
    p: Pair[T -> A, Pair[T -> B, Pair[T -> C, T -> D]]],
    q: Pair[T -> A, Pair[T -> B, Pair[T -> C, T -> D]]],
    r: Pair[T -> A, Pair[T -> B, Pair[T -> C, T -> D]]]
) {
    function_quadruple_mul(function_quadruple_add(p, q), r) =
    function_quadruple_add(function_quadruple_mul(p, r), function_quadruple_mul(q, r))
} by {
    let lhs = function_quadruple_mul(function_quadruple_add(p, q), r)
    let rhs = function_quadruple_add(function_quadruple_mul(p, r), function_quadruple_mul(q, r))
    lhs.first = pointwise_mul(function_quadruple_add(p, q).first, r.first)
    function_quadruple_add(p, q).first = pointwise_add(p.first, q.first)
    lhs.first = pointwise_mul(pointwise_add(p.first, q.first), r.first)
    rhs.first = pointwise_add(function_quadruple_mul(p, r).first, function_quadruple_mul(q, r).first)
    function_quadruple_mul(p, r).first = pointwise_mul(p.first, r.first)
    function_quadruple_mul(q, r).first = pointwise_mul(q.first, r.first)
    rhs.first = pointwise_add(pointwise_mul(p.first, r.first), pointwise_mul(q.first, r.first))
    pointwise_mul_distrib_right(p.first, q.first, r.first)
    lhs.first = rhs.first
    lhs.second.first = pointwise_mul(function_quadruple_add(p, q).second.first, r.second.first)
    function_quadruple_add(p, q).second.first = pointwise_add(p.second.first, q.second.first)
    lhs.second.first = pointwise_mul(pointwise_add(p.second.first, q.second.first), r.second.first)
    rhs.second.first = pointwise_add(
        function_quadruple_mul(p, r).second.first,
        function_quadruple_mul(q, r).second.first)
    function_quadruple_mul(p, r).second.first = pointwise_mul(p.second.first, r.second.first)
    function_quadruple_mul(q, r).second.first = pointwise_mul(q.second.first, r.second.first)
    rhs.second.first =
        pointwise_add(pointwise_mul(p.second.first, r.second.first),
            pointwise_mul(q.second.first, r.second.first))
    pointwise_mul_distrib_right(p.second.first, q.second.first, r.second.first)
    lhs.second.first = rhs.second.first
    lhs.second.second.first = pointwise_mul(
        function_quadruple_add(p, q).second.second.first, r.second.second.first)
    function_quadruple_add(p, q).second.second.first =
        pointwise_add(p.second.second.first, q.second.second.first)
    lhs.second.second.first = pointwise_mul(
        pointwise_add(p.second.second.first, q.second.second.first), r.second.second.first)
    rhs.second.second.first = pointwise_add(
        function_quadruple_mul(p, r).second.second.first,
        function_quadruple_mul(q, r).second.second.first)
    function_quadruple_mul(p, r).second.second.first =
        pointwise_mul(p.second.second.first, r.second.second.first)
    function_quadruple_mul(q, r).second.second.first =
        pointwise_mul(q.second.second.first, r.second.second.first)
    rhs.second.second.first =
        pointwise_add(pointwise_mul(p.second.second.first, r.second.second.first),
            pointwise_mul(q.second.second.first, r.second.second.first))
    pointwise_mul_distrib_right(p.second.second.first, q.second.second.first, r.second.second.first)
    lhs.second.second.first = rhs.second.second.first
    lhs.second.second.second = pointwise_mul(
        function_quadruple_add(p, q).second.second.second, r.second.second.second)
    function_quadruple_add(p, q).second.second.second =
        pointwise_add(p.second.second.second, q.second.second.second)
    lhs.second.second.second = pointwise_mul(
        pointwise_add(p.second.second.second, q.second.second.second), r.second.second.second)
    rhs.second.second.second = pointwise_add(
        function_quadruple_mul(p, r).second.second.second,
        function_quadruple_mul(q, r).second.second.second)
    function_quadruple_mul(p, r).second.second.second =
        pointwise_mul(p.second.second.second, r.second.second.second)
    function_quadruple_mul(q, r).second.second.second =
        pointwise_mul(q.second.second.second, r.second.second.second)
    rhs.second.second.second =
        pointwise_add(pointwise_mul(p.second.second.second, r.second.second.second),
            pointwise_mul(q.second.second.second, r.second.second.second))
    pointwise_mul_distrib_right(p.second.second.second, q.second.second.second, r.second.second.second)
    lhs.second.second.second =
        pointwise_add(pointwise_mul(p.second.second.second, r.second.second.second),
            pointwise_mul(q.second.second.second, r.second.second.second))
    lhs.second.second.second = rhs.second.second.second
    function_quadruple_ext(lhs, rhs)
}

/// Multiplying by the quaternary pointwise zero on the right gives the pointwise zero.
theorem function_quadruple_mul_zero_right[T, A: Semiring, B: Semiring, C: Semiring, D: Semiring](
    p: Pair[T -> A, Pair[T -> B, Pair[T -> C, T -> D]]]
) {
    function_quadruple_mul(p, function_quadruple_zero[T, A, B, C, D]) =
    function_quadruple_zero[T, A, B, C, D]
} by {
    let lhs = function_quadruple_mul(p, function_quadruple_zero[T, A, B, C, D])
    lhs.first = pointwise_mul(p.first, function_quadruple_zero[T, A, B, C, D].first)
    function_quadruple_zero[T, A, B, C, D].first = pointwise_zero[T, A]
    lhs.first = pointwise_mul(p.first, pointwise_zero[T, A])
    pointwise_mul_zero_right(p.first)
    lhs.first = pointwise_zero[T, A]
    lhs.second.first = pointwise_mul(p.second.first,
        function_quadruple_zero[T, A, B, C, D].second.first)
    function_quadruple_zero[T, A, B, C, D].second.first = pointwise_zero[T, B]
    lhs.second.first = pointwise_mul(p.second.first, pointwise_zero[T, B])
    pointwise_mul_zero_right(p.second.first)
    lhs.second.first = pointwise_zero[T, B]
    lhs.second.second.first = pointwise_mul(p.second.second.first,
        function_quadruple_zero[T, A, B, C, D].second.second.first)
    function_quadruple_zero[T, A, B, C, D].second.second.first = pointwise_zero[T, C]
    lhs.second.second.first = pointwise_mul(p.second.second.first, pointwise_zero[T, C])
    pointwise_mul_zero_right(p.second.second.first)
    lhs.second.second.first = pointwise_zero[T, C]
    lhs.second.second.second = pointwise_mul(p.second.second.second,
        function_quadruple_zero[T, A, B, C, D].second.second.second)
    function_quadruple_zero[T, A, B, C, D].second.second.second = pointwise_zero[T, D]
    lhs.second.second.second = pointwise_mul(p.second.second.second, pointwise_zero[T, D])
    pointwise_mul_zero_right(p.second.second.second)
    lhs.second.second.second = pointwise_zero[T, D]
    function_quadruple_ext(lhs, function_quadruple_zero[T, A, B, C, D])
}

/// Multiplying by the quaternary pointwise zero on the left gives the pointwise zero.
theorem function_quadruple_mul_zero_left[T, A: Semiring, B: Semiring, C: Semiring, D: Semiring](
    p: Pair[T -> A, Pair[T -> B, Pair[T -> C, T -> D]]]
) {
    function_quadruple_mul(function_quadruple_zero[T, A, B, C, D], p) =
    function_quadruple_zero[T, A, B, C, D]
} by {
    let lhs = function_quadruple_mul(function_quadruple_zero[T, A, B, C, D], p)
    lhs.first = pointwise_mul(function_quadruple_zero[T, A, B, C, D].first, p.first)
    function_quadruple_zero[T, A, B, C, D].first = pointwise_zero[T, A]
    lhs.first = pointwise_mul(pointwise_zero[T, A], p.first)
    pointwise_mul_zero_left(p.first)
    lhs.first = pointwise_zero[T, A]
    lhs.second.first = pointwise_mul(
        function_quadruple_zero[T, A, B, C, D].second.first, p.second.first)
    function_quadruple_zero[T, A, B, C, D].second.first = pointwise_zero[T, B]
    lhs.second.first = pointwise_mul(pointwise_zero[T, B], p.second.first)
    pointwise_mul_zero_left(p.second.first)
    lhs.second.first = pointwise_zero[T, B]
    lhs.second.second.first = pointwise_mul(
        function_quadruple_zero[T, A, B, C, D].second.second.first, p.second.second.first)
    function_quadruple_zero[T, A, B, C, D].second.second.first = pointwise_zero[T, C]
    lhs.second.second.first = pointwise_mul(pointwise_zero[T, C], p.second.second.first)
    pointwise_mul_zero_left(p.second.second.first)
    lhs.second.second.first = pointwise_zero[T, C]
    lhs.second.second.second = pointwise_mul(
        function_quadruple_zero[T, A, B, C, D].second.second.second, p.second.second.second)
    function_quadruple_zero[T, A, B, C, D].second.second.second = pointwise_zero[T, D]
    lhs.second.second.second = pointwise_mul(pointwise_zero[T, D], p.second.second.second)
    pointwise_mul_zero_left(p.second.second.second)
    lhs.second.second.second = pointwise_zero[T, D]
    function_quadruple_ext(lhs, function_quadruple_zero[T, A, B, C, D])
}

/// The value of a quinary product of functions at one point.
define function_quintuple_apply[T, A, B, C, D, E](
    p: Pair[T -> A, Pair[T -> B, Pair[T -> C, Pair[T -> D, T -> E]]]],
    t: T
) -> Pair[A, Pair[B, Pair[C, Pair[D, E]]]] {
    Pair.new(p.first(t), function_quadruple_apply(p.second, t))
}

/// The quinary product of five functions.
define function_quintuple[T, A, B, C, D, E](
    f: T -> A,
    g: T -> B,
    h: T -> C,
    i: T -> D,
    j: T -> E
) -> Pair[T -> A, Pair[T -> B, Pair[T -> C, Pair[T -> D, T -> E]]]] {
    Pair.new(f, function_quadruple(g, h, i, j))
}

/// The pointwise sum on a quinary product of function spaces.
define function_quintuple_add[T, A: Add, B: Add, C: Add, D: Add, E: Add](
    p: Pair[T -> A, Pair[T -> B, Pair[T -> C, Pair[T -> D, T -> E]]]],
    q: Pair[T -> A, Pair[T -> B, Pair[T -> C, Pair[T -> D, T -> E]]]]
) -> Pair[T -> A, Pair[T -> B, Pair[T -> C, Pair[T -> D, T -> E]]]] {
    Pair.new(pointwise_add(p.first, q.first), function_quadruple_add(p.second, q.second))
}

/// The pointwise product on a quinary product of function spaces.
define function_quintuple_mul[T, A: Mul, B: Mul, C: Mul, D: Mul, E: Mul](
    p: Pair[T -> A, Pair[T -> B, Pair[T -> C, Pair[T -> D, T -> E]]]],
    q: Pair[T -> A, Pair[T -> B, Pair[T -> C, Pair[T -> D, T -> E]]]]
) -> Pair[T -> A, Pair[T -> B, Pair[T -> C, Pair[T -> D, T -> E]]]] {
    Pair.new(pointwise_mul(p.first, q.first), function_quadruple_mul(p.second, q.second))
}

/// The pointwise zero on a quinary product of function spaces.
let function_quintuple_zero[T, A: Zero, B: Zero, C: Zero, D: Zero, E: Zero]:
    Pair[T -> A, Pair[T -> B, Pair[T -> C, Pair[T -> D, T -> E]]]] =
    Pair.new(pointwise_zero[T, A], function_quadruple_zero[T, B, C, D, E])

/// The pointwise one on a quinary product of function spaces.
let function_quintuple_one[T, A: One, B: One, C: One, D: One, E: One]:
    Pair[T -> A, Pair[T -> B, Pair[T -> C, Pair[T -> D, T -> E]]]] =
    Pair.new(pointwise_one[T, A], function_quadruple_one[T, B, C, D, E])

/// The pointwise negation on a quinary product of function spaces.
define function_quintuple_neg[T, A: Neg, B: Neg, C: Neg, D: Neg, E: Neg](
    p: Pair[T -> A, Pair[T -> B, Pair[T -> C, Pair[T -> D, T -> E]]]]
) -> Pair[T -> A, Pair[T -> B, Pair[T -> C, Pair[T -> D, T -> E]]]] {
    Pair.new(pointwise_neg(p.first), function_quadruple_neg(p.second))
}

/// The pointwise inverse on a quinary product of function spaces.
define function_quintuple_inverse[T, A: Group, B: Group, C: Group, D: Group, E: Group](
    p: Pair[T -> A, Pair[T -> B, Pair[T -> C, Pair[T -> D, T -> E]]]]
) -> Pair[T -> A, Pair[T -> B, Pair[T -> C, Pair[T -> D, T -> E]]]] {
    Pair.new(pointwise_inverse(p.first), function_quadruple_inverse(p.second))
}

/// Evaluation of a quinary product has the expected first coordinate.
theorem function_quintuple_apply_first[T, A, B, C, D, E](
    p: Pair[T -> A, Pair[T -> B, Pair[T -> C, Pair[T -> D, T -> E]]]],
    t: T
) {
    function_quintuple_apply(p, t).first = p.first(t)
}

/// Evaluation of a quinary product has the expected second coordinate.
theorem function_quintuple_apply_second_first[T, A, B, C, D, E](
    p: Pair[T -> A, Pair[T -> B, Pair[T -> C, Pair[T -> D, T -> E]]]],
    t: T
) {
    function_quintuple_apply(p, t).second.first = p.second.first(t)
} by {
    function_quintuple_apply(p, t).second = function_quadruple_apply(p.second, t)
    function_quadruple_apply_first(p.second, t)
}

/// Evaluation of a quinary product has the expected third coordinate.
theorem function_quintuple_apply_second_second_first[T, A, B, C, D, E](
    p: Pair[T -> A, Pair[T -> B, Pair[T -> C, Pair[T -> D, T -> E]]]],
    t: T
) {
    function_quintuple_apply(p, t).second.second.first = p.second.second.first(t)
} by {
    function_quintuple_apply(p, t).second = function_quadruple_apply(p.second, t)
    function_quadruple_apply_second_first(p.second, t)
}

/// Evaluation of a quinary product has the expected fourth coordinate.
theorem function_quintuple_apply_second_second_second_first[T, A, B, C, D, E](
    p: Pair[T -> A, Pair[T -> B, Pair[T -> C, Pair[T -> D, T -> E]]]],
    t: T
) {
    function_quintuple_apply(p, t).second.second.second.first =
    p.second.second.second.first(t)
} by {
    function_quintuple_apply(p, t).second = function_quadruple_apply(p.second, t)
    function_quadruple_apply_second_second_first(p.second, t)
}

/// Evaluation of a quinary product has the expected fifth coordinate.
theorem function_quintuple_apply_second_second_second_second[T, A, B, C, D, E](
    p: Pair[T -> A, Pair[T -> B, Pair[T -> C, Pair[T -> D, T -> E]]]],
    t: T
) {
    function_quintuple_apply(p, t).second.second.second.second =
    p.second.second.second.second(t)
} by {
    function_quintuple_apply(p, t).second = function_quadruple_apply(p.second, t)
    function_quadruple_apply_second_second_second(p.second, t)
}

/// The quinary product of five functions has the expected first coordinate.
theorem function_quintuple_first[T, A, B, C, D, E](
    f: T -> A, g: T -> B, h: T -> C, i: T -> D, j: T -> E
) {
    function_quintuple(f, g, h, i, j).first = f
}

/// The quinary product of five functions has the expected second coordinate.
theorem function_quintuple_second_first[T, A, B, C, D, E](
    f: T -> A, g: T -> B, h: T -> C, i: T -> D, j: T -> E
) {
    function_quintuple(f, g, h, i, j).second.first = g
} by {
    function_quintuple(f, g, h, i, j).second = function_quadruple(g, h, i, j)
    function_quadruple_first(g, h, i, j)
}

/// The quinary product of five functions has the expected third coordinate.
theorem function_quintuple_second_second_first[T, A, B, C, D, E](
    f: T -> A, g: T -> B, h: T -> C, i: T -> D, j: T -> E
) {
    function_quintuple(f, g, h, i, j).second.second.first = h
} by {
    function_quintuple(f, g, h, i, j).second = function_quadruple(g, h, i, j)
    function_quadruple_second_first(g, h, i, j)
}

/// The quinary product of five functions has the expected fourth coordinate.
theorem function_quintuple_second_second_second_first[T, A, B, C, D, E](
    f: T -> A, g: T -> B, h: T -> C, i: T -> D, j: T -> E
) {
    function_quintuple(f, g, h, i, j).second.second.second.first = i
} by {
    function_quintuple(f, g, h, i, j).second = function_quadruple(g, h, i, j)
    function_quadruple_second_second_first(g, h, i, j)
}

/// The quinary product of five functions has the expected fifth coordinate.
theorem function_quintuple_second_second_second_second[T, A, B, C, D, E](
    f: T -> A, g: T -> B, h: T -> C, i: T -> D, j: T -> E
) {
    function_quintuple(f, g, h, i, j).second.second.second.second = j
} by {
    function_quintuple(f, g, h, i, j).second = function_quadruple(g, h, i, j)
    function_quadruple_second_second_second(g, h, i, j)
}

/// Function quintuples are determined by their five coordinates.
theorem function_quintuple_eta[T, A, B, C, D, E](
    p: Pair[T -> A, Pair[T -> B, Pair[T -> C, Pair[T -> D, T -> E]]]]
) {
    function_quintuple(p.first, p.second.first, p.second.second.first,
        p.second.second.second.first, p.second.second.second.second) = p
} by {
    let q = function_quintuple(p.first, p.second.first, p.second.second.first,
        p.second.second.second.first, p.second.second.second.second)
    q.first = p.first
    q.second = function_quadruple(p.second.first, p.second.second.first,
        p.second.second.second.first, p.second.second.second.second)
    function_quadruple_eta(p.second)
    q.second = p.second
    pair_ext(q, p)
}

/// Quinary function products are determined by their five coordinates.
theorem function_quintuple_ext[T, A, B, C, D, E](
    p: Pair[T -> A, Pair[T -> B, Pair[T -> C, Pair[T -> D, T -> E]]]],
    q: Pair[T -> A, Pair[T -> B, Pair[T -> C, Pair[T -> D, T -> E]]]]
) {
    p.first = q.first and p.second.first = q.second.first and
    p.second.second.first = q.second.second.first and
    p.second.second.second.first = q.second.second.second.first and
    p.second.second.second.second = q.second.second.second.second implies p = q
} by {
    if p.first = q.first and p.second.first = q.second.first and
    p.second.second.first = q.second.second.first and
    p.second.second.second.first = q.second.second.second.first and
    p.second.second.second.second = q.second.second.second.second {
        function_quadruple_ext(p.second, q.second)
        p.second = q.second
        pair_ext(p, q)
    }
}

/// The first coordinate of a quinary pointwise sum is the pointwise sum of first coordinates.
theorem function_quintuple_add_first[T, A: Add, B: Add, C: Add, D: Add, E: Add](
    p: Pair[T -> A, Pair[T -> B, Pair[T -> C, Pair[T -> D, T -> E]]]],
    q: Pair[T -> A, Pair[T -> B, Pair[T -> C, Pair[T -> D, T -> E]]]]
) {
    function_quintuple_add(p, q).first = pointwise_add(p.first, q.first)
} by {
    function_quintuple_add(p, q) =
        Pair.new(pointwise_add(p.first, q.first), function_quadruple_add(p.second, q.second))
}

/// The second coordinate of a quinary pointwise sum is the pointwise sum of second coordinates.
theorem function_quintuple_add_second_first[T, A: Add, B: Add, C: Add, D: Add, E: Add](
    p: Pair[T -> A, Pair[T -> B, Pair[T -> C, Pair[T -> D, T -> E]]]],
    q: Pair[T -> A, Pair[T -> B, Pair[T -> C, Pair[T -> D, T -> E]]]]
) {
    function_quintuple_add(p, q).second.first =
        pointwise_add(p.second.first, q.second.first)
} by {
    function_quintuple_add(p, q).second = function_quadruple_add(p.second, q.second)
    function_quadruple_add_first(p.second, q.second)
}

/// The third coordinate of a quinary pointwise sum is the pointwise sum of third coordinates.
theorem function_quintuple_add_second_second_first[T, A: Add, B: Add, C: Add, D: Add, E: Add](
    p: Pair[T -> A, Pair[T -> B, Pair[T -> C, Pair[T -> D, T -> E]]]],
    q: Pair[T -> A, Pair[T -> B, Pair[T -> C, Pair[T -> D, T -> E]]]]
) {
    function_quintuple_add(p, q).second.second.first =
        pointwise_add(p.second.second.first, q.second.second.first)
} by {
    function_quintuple_add(p, q).second = function_quadruple_add(p.second, q.second)
    function_quadruple_add_second_first(p.second, q.second)
}

/// The fourth coordinate of a quinary pointwise sum is the pointwise sum of fourth coordinates.
theorem function_quintuple_add_second_second_second_first[T, A: Add, B: Add, C: Add, D: Add, E: Add](
    p: Pair[T -> A, Pair[T -> B, Pair[T -> C, Pair[T -> D, T -> E]]]],
    q: Pair[T -> A, Pair[T -> B, Pair[T -> C, Pair[T -> D, T -> E]]]]
) {
    function_quintuple_add(p, q).second.second.second.first =
        pointwise_add(p.second.second.second.first, q.second.second.second.first)
} by {
    function_quintuple_add(p, q).second = function_quadruple_add(p.second, q.second)
    function_quadruple_add_second_second_first(p.second, q.second)
}

/// The fifth coordinate of a quinary pointwise sum is the pointwise sum of fifth coordinates.
theorem function_quintuple_add_second_second_second_second[T, A: Add, B: Add, C: Add, D: Add, E: Add](
    p: Pair[T -> A, Pair[T -> B, Pair[T -> C, Pair[T -> D, T -> E]]]],
    q: Pair[T -> A, Pair[T -> B, Pair[T -> C, Pair[T -> D, T -> E]]]]
) {
    function_quintuple_add(p, q).second.second.second.second =
        pointwise_add(p.second.second.second.second, q.second.second.second.second)
} by {
    function_quintuple_add(p, q).second = function_quadruple_add(p.second, q.second)
    function_quadruple_add_second_second_second(p.second, q.second)
}

/// The first coordinate of a quinary pointwise product is the pointwise product of first coordinates.
theorem function_quintuple_mul_first[T, A: Mul, B: Mul, C: Mul, D: Mul, E: Mul](
    p: Pair[T -> A, Pair[T -> B, Pair[T -> C, Pair[T -> D, T -> E]]]],
    q: Pair[T -> A, Pair[T -> B, Pair[T -> C, Pair[T -> D, T -> E]]]]
) {
    function_quintuple_mul(p, q).first = pointwise_mul(p.first, q.first)
} by {
    function_quintuple_mul(p, q) =
        Pair.new(pointwise_mul(p.first, q.first), function_quadruple_mul(p.second, q.second))
}

/// The second coordinate of a quinary pointwise product is the pointwise product of second coordinates.
theorem function_quintuple_mul_second_first[T, A: Mul, B: Mul, C: Mul, D: Mul, E: Mul](
    p: Pair[T -> A, Pair[T -> B, Pair[T -> C, Pair[T -> D, T -> E]]]],
    q: Pair[T -> A, Pair[T -> B, Pair[T -> C, Pair[T -> D, T -> E]]]]
) {
    function_quintuple_mul(p, q).second.first =
        pointwise_mul(p.second.first, q.second.first)
} by {
    function_quintuple_mul(p, q).second = function_quadruple_mul(p.second, q.second)
    function_quadruple_mul_first(p.second, q.second)
}

/// The third coordinate of a quinary pointwise product is the pointwise product of third coordinates.
theorem function_quintuple_mul_second_second_first[T, A: Mul, B: Mul, C: Mul, D: Mul, E: Mul](
    p: Pair[T -> A, Pair[T -> B, Pair[T -> C, Pair[T -> D, T -> E]]]],
    q: Pair[T -> A, Pair[T -> B, Pair[T -> C, Pair[T -> D, T -> E]]]]
) {
    function_quintuple_mul(p, q).second.second.first =
        pointwise_mul(p.second.second.first, q.second.second.first)
} by {
    function_quintuple_mul(p, q).second = function_quadruple_mul(p.second, q.second)
    function_quadruple_mul_second_first(p.second, q.second)
}

/// The fourth coordinate of a quinary pointwise product is the pointwise product of fourth coordinates.
theorem function_quintuple_mul_second_second_second_first[T, A: Mul, B: Mul, C: Mul, D: Mul, E: Mul](
    p: Pair[T -> A, Pair[T -> B, Pair[T -> C, Pair[T -> D, T -> E]]]],
    q: Pair[T -> A, Pair[T -> B, Pair[T -> C, Pair[T -> D, T -> E]]]]
) {
    function_quintuple_mul(p, q).second.second.second.first =
        pointwise_mul(p.second.second.second.first, q.second.second.second.first)
} by {
    function_quintuple_mul(p, q).second = function_quadruple_mul(p.second, q.second)
    function_quadruple_mul_second_second_first(p.second, q.second)
}

/// The fifth coordinate of a quinary pointwise product is the pointwise product of fifth coordinates.
theorem function_quintuple_mul_second_second_second_second[T, A: Mul, B: Mul, C: Mul, D: Mul, E: Mul](
    p: Pair[T -> A, Pair[T -> B, Pair[T -> C, Pair[T -> D, T -> E]]]],
    q: Pair[T -> A, Pair[T -> B, Pair[T -> C, Pair[T -> D, T -> E]]]]
) {
    function_quintuple_mul(p, q).second.second.second.second =
        pointwise_mul(p.second.second.second.second, q.second.second.second.second)
} by {
    function_quintuple_mul(p, q).second = function_quadruple_mul(p.second, q.second)
    function_quadruple_mul_second_second_second(p.second, q.second)
}

/// The first coordinate of the quinary pointwise zero is the pointwise zero.
theorem function_quintuple_zero_first[T, A: Zero, B: Zero, C: Zero, D: Zero, E: Zero] {
    function_quintuple_zero[T, A, B, C, D, E].first = pointwise_zero[T, A]
} by {
    function_quintuple_zero[T, A, B, C, D, E] =
        Pair.new(pointwise_zero[T, A], function_quadruple_zero[T, B, C, D, E])
}

/// The second coordinate of the quinary pointwise zero is the pointwise zero.
theorem function_quintuple_zero_second_first[T, A: Zero, B: Zero, C: Zero, D: Zero, E: Zero] {
    function_quintuple_zero[T, A, B, C, D, E].second.first = pointwise_zero[T, B]
} by {
    function_quintuple_zero[T, A, B, C, D, E].second =
        function_quadruple_zero[T, B, C, D, E]
    function_quadruple_zero_first[T, B, C, D, E]
}

/// The third coordinate of the quinary pointwise zero is the pointwise zero.
theorem function_quintuple_zero_second_second_first[T, A: Zero, B: Zero, C: Zero, D: Zero, E: Zero] {
    function_quintuple_zero[T, A, B, C, D, E].second.second.first =
        pointwise_zero[T, C]
} by {
    function_quintuple_zero[T, A, B, C, D, E].second =
        function_quadruple_zero[T, B, C, D, E]
    function_quadruple_zero_second_first[T, B, C, D, E]
}

/// The fourth coordinate of the quinary pointwise zero is the pointwise zero.
theorem function_quintuple_zero_second_second_second_first[T, A: Zero, B: Zero, C: Zero, D: Zero, E: Zero] {
    function_quintuple_zero[T, A, B, C, D, E].second.second.second.first =
        pointwise_zero[T, D]
} by {
    function_quintuple_zero[T, A, B, C, D, E].second =
        function_quadruple_zero[T, B, C, D, E]
    function_quadruple_zero_second_second_first[T, B, C, D, E]
}

/// The fifth coordinate of the quinary pointwise zero is the pointwise zero.
theorem function_quintuple_zero_second_second_second_second[T, A: Zero, B: Zero, C: Zero, D: Zero, E: Zero] {
    function_quintuple_zero[T, A, B, C, D, E].second.second.second.second =
        pointwise_zero[T, E]
} by {
    function_quintuple_zero[T, A, B, C, D, E].second =
        function_quadruple_zero[T, B, C, D, E]
    function_quadruple_zero_second_second_second[T, B, C, D, E]
}

/// The first coordinate of the quinary pointwise one is the pointwise one.
theorem function_quintuple_one_first[T, A: One, B: One, C: One, D: One, E: One] {
    function_quintuple_one[T, A, B, C, D, E].first = pointwise_one[T, A]
} by {
    function_quintuple_one[T, A, B, C, D, E] =
        Pair.new(pointwise_one[T, A], function_quadruple_one[T, B, C, D, E])
}

/// The second coordinate of the quinary pointwise one is the pointwise one.
theorem function_quintuple_one_second_first[T, A: One, B: One, C: One, D: One, E: One] {
    function_quintuple_one[T, A, B, C, D, E].second.first = pointwise_one[T, B]
} by {
    function_quintuple_one[T, A, B, C, D, E].second =
        function_quadruple_one[T, B, C, D, E]
    function_quadruple_one_first[T, B, C, D, E]
}

/// The third coordinate of the quinary pointwise one is the pointwise one.
theorem function_quintuple_one_second_second_first[T, A: One, B: One, C: One, D: One, E: One] {
    function_quintuple_one[T, A, B, C, D, E].second.second.first =
        pointwise_one[T, C]
} by {
    function_quintuple_one[T, A, B, C, D, E].second =
        function_quadruple_one[T, B, C, D, E]
    function_quadruple_one_second_first[T, B, C, D, E]
}

/// The fourth coordinate of the quinary pointwise one is the pointwise one.
theorem function_quintuple_one_second_second_second_first[T, A: One, B: One, C: One, D: One, E: One] {
    function_quintuple_one[T, A, B, C, D, E].second.second.second.first =
        pointwise_one[T, D]
} by {
    function_quintuple_one[T, A, B, C, D, E].second =
        function_quadruple_one[T, B, C, D, E]
    function_quadruple_one_second_second_first[T, B, C, D, E]
}

/// The fifth coordinate of the quinary pointwise one is the pointwise one.
theorem function_quintuple_one_second_second_second_second[T, A: One, B: One, C: One, D: One, E: One] {
    function_quintuple_one[T, A, B, C, D, E].second.second.second.second =
        pointwise_one[T, E]
} by {
    function_quintuple_one[T, A, B, C, D, E].second =
        function_quadruple_one[T, B, C, D, E]
    function_quadruple_one_second_second_second[T, B, C, D, E]
}

/// The first coordinate of quinary pointwise negation is pointwise negation.
theorem function_quintuple_neg_first[T, A: Neg, B: Neg, C: Neg, D: Neg, E: Neg](
    p: Pair[T -> A, Pair[T -> B, Pair[T -> C, Pair[T -> D, T -> E]]]]
) {
    function_quintuple_neg(p).first = pointwise_neg(p.first)
} by {
    function_quintuple_neg(p) = Pair.new(pointwise_neg(p.first), function_quadruple_neg(p.second))
}

/// The second coordinate of quinary pointwise negation is pointwise negation.
theorem function_quintuple_neg_second_first[T, A: Neg, B: Neg, C: Neg, D: Neg, E: Neg](
    p: Pair[T -> A, Pair[T -> B, Pair[T -> C, Pair[T -> D, T -> E]]]]
) {
    function_quintuple_neg(p).second.first = pointwise_neg(p.second.first)
} by {
    function_quintuple_neg(p).second = function_quadruple_neg(p.second)
    function_quadruple_neg_first(p.second)
}

/// The third coordinate of quinary pointwise negation is pointwise negation.
theorem function_quintuple_neg_second_second_first[T, A: Neg, B: Neg, C: Neg, D: Neg, E: Neg](
    p: Pair[T -> A, Pair[T -> B, Pair[T -> C, Pair[T -> D, T -> E]]]]
) {
    function_quintuple_neg(p).second.second.first = pointwise_neg(p.second.second.first)
} by {
    function_quintuple_neg(p).second = function_quadruple_neg(p.second)
    function_quadruple_neg_second_first(p.second)
}

/// The fourth coordinate of quinary pointwise negation is pointwise negation.
theorem function_quintuple_neg_second_second_second_first[T, A: Neg, B: Neg, C: Neg, D: Neg, E: Neg](
    p: Pair[T -> A, Pair[T -> B, Pair[T -> C, Pair[T -> D, T -> E]]]]
) {
    function_quintuple_neg(p).second.second.second.first =
        pointwise_neg(p.second.second.second.first)
} by {
    function_quintuple_neg(p).second = function_quadruple_neg(p.second)
    function_quadruple_neg_second_second_first(p.second)
}

/// The fifth coordinate of quinary pointwise negation is pointwise negation.
theorem function_quintuple_neg_second_second_second_second[T, A: Neg, B: Neg, C: Neg, D: Neg, E: Neg](
    p: Pair[T -> A, Pair[T -> B, Pair[T -> C, Pair[T -> D, T -> E]]]]
) {
    function_quintuple_neg(p).second.second.second.second =
        pointwise_neg(p.second.second.second.second)
} by {
    function_quintuple_neg(p).second = function_quadruple_neg(p.second)
    function_quadruple_neg_second_second_second(p.second)
}

/// The first coordinate of quinary pointwise inversion is pointwise inversion.
theorem function_quintuple_inverse_first[T, A: Group, B: Group, C: Group, D: Group, E: Group](
    p: Pair[T -> A, Pair[T -> B, Pair[T -> C, Pair[T -> D, T -> E]]]]
) {
    function_quintuple_inverse(p).first = pointwise_inverse(p.first)
} by {
    function_quintuple_inverse(p) =
        Pair.new(pointwise_inverse(p.first), function_quadruple_inverse(p.second))
}

/// The second coordinate of quinary pointwise inversion is pointwise inversion.
theorem function_quintuple_inverse_second_first[T, A: Group, B: Group, C: Group, D: Group, E: Group](
    p: Pair[T -> A, Pair[T -> B, Pair[T -> C, Pair[T -> D, T -> E]]]]
) {
    function_quintuple_inverse(p).second.first = pointwise_inverse(p.second.first)
} by {
    function_quintuple_inverse(p).second = function_quadruple_inverse(p.second)
    function_quadruple_inverse_first(p.second)
}

/// The third coordinate of quinary pointwise inversion is pointwise inversion.
theorem function_quintuple_inverse_second_second_first[T, A: Group, B: Group, C: Group, D: Group, E: Group](
    p: Pair[T -> A, Pair[T -> B, Pair[T -> C, Pair[T -> D, T -> E]]]]
) {
    function_quintuple_inverse(p).second.second.first =
        pointwise_inverse(p.second.second.first)
} by {
    function_quintuple_inverse(p).second = function_quadruple_inverse(p.second)
    function_quadruple_inverse_second_first(p.second)
}

/// The fourth coordinate of quinary pointwise inversion is pointwise inversion.
theorem function_quintuple_inverse_second_second_second_first[T, A: Group, B: Group, C: Group, D: Group, E: Group](
    p: Pair[T -> A, Pair[T -> B, Pair[T -> C, Pair[T -> D, T -> E]]]]
) {
    function_quintuple_inverse(p).second.second.second.first =
        pointwise_inverse(p.second.second.second.first)
} by {
    function_quintuple_inverse(p).second = function_quadruple_inverse(p.second)
    function_quadruple_inverse_second_second_first(p.second)
}

/// The fifth coordinate of quinary pointwise inversion is pointwise inversion.
theorem function_quintuple_inverse_second_second_second_second[T, A: Group, B: Group, C: Group, D: Group, E: Group](
    p: Pair[T -> A, Pair[T -> B, Pair[T -> C, Pair[T -> D, T -> E]]]]
) {
    function_quintuple_inverse(p).second.second.second.second =
        pointwise_inverse(p.second.second.second.second)
} by {
    function_quintuple_inverse(p).second = function_quadruple_inverse(p.second)
    function_quadruple_inverse_second_second_second(p.second)
}

/// Pointwise addition is associative on a quinary product of function spaces.
theorem function_quintuple_add_assoc[T, A: AddSemigroup, B: AddSemigroup, C: AddSemigroup, D: AddSemigroup, E: AddSemigroup](
    p: Pair[T -> A, Pair[T -> B, Pair[T -> C, Pair[T -> D, T -> E]]]],
    q: Pair[T -> A, Pair[T -> B, Pair[T -> C, Pair[T -> D, T -> E]]]],
    r: Pair[T -> A, Pair[T -> B, Pair[T -> C, Pair[T -> D, T -> E]]]]
) {
    function_quintuple_add(p, function_quintuple_add(q, r)) =
    function_quintuple_add(function_quintuple_add(p, q), r)
} by {
    let lhs = function_quintuple_add(p, function_quintuple_add(q, r))
    let rhs = function_quintuple_add(function_quintuple_add(p, q), r)
    lhs.first = pointwise_add(p.first, function_quintuple_add(q, r).first)
    function_quintuple_add(q, r).first = pointwise_add(q.first, r.first)
    lhs.first = pointwise_add(p.first, pointwise_add(q.first, r.first))
    rhs.first = pointwise_add(function_quintuple_add(p, q).first, r.first)
    function_quintuple_add(p, q).first = pointwise_add(p.first, q.first)
    rhs.first = pointwise_add(pointwise_add(p.first, q.first), r.first)
    pointwise_add_assoc(p.first, q.first, r.first)
    lhs.first = rhs.first
    lhs.second = function_quadruple_add(p.second, function_quintuple_add(q, r).second)
    function_quintuple_add(q, r).second = function_quadruple_add(q.second, r.second)
    lhs.second = function_quadruple_add(p.second, function_quadruple_add(q.second, r.second))
    rhs.second = function_quadruple_add(function_quintuple_add(p, q).second, r.second)
    function_quintuple_add(p, q).second = function_quadruple_add(p.second, q.second)
    rhs.second = function_quadruple_add(function_quadruple_add(p.second, q.second), r.second)
    function_quadruple_add_assoc(p.second, q.second, r.second)
    lhs.second = rhs.second
    pair_ext(lhs, rhs)
}

/// Pointwise addition is commutative on a quinary product of function spaces.
theorem function_quintuple_add_comm[T, A: AddCommSemigroup, B: AddCommSemigroup, C: AddCommSemigroup, D: AddCommSemigroup, E: AddCommSemigroup](
    p: Pair[T -> A, Pair[T -> B, Pair[T -> C, Pair[T -> D, T -> E]]]],
    q: Pair[T -> A, Pair[T -> B, Pair[T -> C, Pair[T -> D, T -> E]]]]
) {
    function_quintuple_add(p, q) = function_quintuple_add(q, p)
} by {
    let lhs = function_quintuple_add(p, q)
    let rhs = function_quintuple_add(q, p)
    lhs.first = pointwise_add(p.first, q.first)
    rhs.first = pointwise_add(q.first, p.first)
    pointwise_add_comm(p.first, q.first)
    lhs.first = rhs.first
    lhs.second = function_quadruple_add(p.second, q.second)
    rhs.second = function_quadruple_add(q.second, p.second)
    function_quadruple_add_comm(p.second, q.second)
    lhs.second = rhs.second
    pair_ext(lhs, rhs)
}

/// Adding the quinary pointwise zero on the right changes no coordinate.
theorem function_quintuple_add_zero_right[T, A: AddMonoid, B: AddMonoid, C: AddMonoid, D: AddMonoid, E: AddMonoid](
    p: Pair[T -> A, Pair[T -> B, Pair[T -> C, Pair[T -> D, T -> E]]]]
) {
    function_quintuple_add(p, function_quintuple_zero[T, A, B, C, D, E]) = p
} by {
    let lhs = function_quintuple_add(p, function_quintuple_zero[T, A, B, C, D, E])
    lhs.first = pointwise_add(p.first, function_quintuple_zero[T, A, B, C, D, E].first)
    function_quintuple_zero[T, A, B, C, D, E].first = pointwise_zero[T, A]
    lhs.first = pointwise_add(p.first, pointwise_zero[T, A])
    pointwise_add_zero_right(p.first)
    lhs.first = p.first
    lhs.second = function_quadruple_add(p.second, function_quintuple_zero[T, A, B, C, D, E].second)
    function_quintuple_zero[T, A, B, C, D, E].second = function_quadruple_zero[T, B, C, D, E]
    lhs.second = function_quadruple_add(p.second, function_quadruple_zero[T, B, C, D, E])
    function_quadruple_add_zero_right(p.second)
    lhs.second = p.second
    pair_ext(lhs, p)
}

/// Adding the quinary pointwise zero on the left changes no coordinate.
theorem function_quintuple_add_zero_left[T, A: AddMonoid, B: AddMonoid, C: AddMonoid, D: AddMonoid, E: AddMonoid](
    p: Pair[T -> A, Pair[T -> B, Pair[T -> C, Pair[T -> D, T -> E]]]]
) {
    function_quintuple_add(function_quintuple_zero[T, A, B, C, D, E], p) = p
} by {
    let lhs = function_quintuple_add(function_quintuple_zero[T, A, B, C, D, E], p)
    lhs.first = pointwise_add(function_quintuple_zero[T, A, B, C, D, E].first, p.first)
    function_quintuple_zero[T, A, B, C, D, E].first = pointwise_zero[T, A]
    lhs.first = pointwise_add(pointwise_zero[T, A], p.first)
    pointwise_add_zero_left(p.first)
    lhs.first = p.first
    lhs.second = function_quadruple_add(function_quintuple_zero[T, A, B, C, D, E].second, p.second)
    function_quintuple_zero[T, A, B, C, D, E].second = function_quadruple_zero[T, B, C, D, E]
    lhs.second = function_quadruple_add(function_quadruple_zero[T, B, C, D, E], p.second)
    function_quadruple_add_zero_left(p.second)
    lhs.second = p.second
    pair_ext(lhs, p)
}

/// Pointwise negation is a right additive inverse on a quinary product of function spaces.
theorem function_quintuple_add_neg_right[T, A: AddGroup, B: AddGroup, C: AddGroup, D: AddGroup, E: AddGroup](
    p: Pair[T -> A, Pair[T -> B, Pair[T -> C, Pair[T -> D, T -> E]]]]
) {
    function_quintuple_add(p, function_quintuple_neg(p)) =
    function_quintuple_zero[T, A, B, C, D, E]
} by {
    let lhs = function_quintuple_add(p, function_quintuple_neg(p))
    let rhs = function_quintuple_zero[T, A, B, C, D, E]
    lhs.first = pointwise_add(p.first, function_quintuple_neg(p).first)
    function_quintuple_neg(p).first = pointwise_neg(p.first)
    lhs.first = pointwise_add(p.first, pointwise_neg(p.first))
    pointwise_add_neg_right(p.first)
    lhs.first = pointwise_zero[T, A]
    rhs.first = pointwise_zero[T, A]
    lhs.second = function_quadruple_add(p.second, function_quintuple_neg(p).second)
    function_quintuple_neg(p).second = function_quadruple_neg(p.second)
    lhs.second = function_quadruple_add(p.second, function_quadruple_neg(p.second))
    function_quadruple_add_neg_right(p.second)
    lhs.second = function_quadruple_zero[T, B, C, D, E]
    rhs.second = function_quadruple_zero[T, B, C, D, E]
    pair_ext(lhs, rhs)
}

/// Pointwise negation is a left additive inverse on a quinary product of function spaces.
theorem function_quintuple_add_neg_left[T, A: AddGroup, B: AddGroup, C: AddGroup, D: AddGroup, E: AddGroup](
    p: Pair[T -> A, Pair[T -> B, Pair[T -> C, Pair[T -> D, T -> E]]]]
) {
    function_quintuple_add(function_quintuple_neg(p), p) =
    function_quintuple_zero[T, A, B, C, D, E]
} by {
    let lhs = function_quintuple_add(function_quintuple_neg(p), p)
    let rhs = function_quintuple_zero[T, A, B, C, D, E]
    lhs.first = pointwise_add(function_quintuple_neg(p).first, p.first)
    function_quintuple_neg(p).first = pointwise_neg(p.first)
    lhs.first = pointwise_add(pointwise_neg(p.first), p.first)
    pointwise_add_neg_left(p.first)
    lhs.first = pointwise_zero[T, A]
    rhs.first = pointwise_zero[T, A]
    lhs.second = function_quadruple_add(function_quintuple_neg(p).second, p.second)
    function_quintuple_neg(p).second = function_quadruple_neg(p.second)
    lhs.second = function_quadruple_add(function_quadruple_neg(p.second), p.second)
    function_quadruple_add_neg_left(p.second)
    lhs.second = function_quadruple_zero[T, B, C, D, E]
    rhs.second = function_quadruple_zero[T, B, C, D, E]
    pair_ext(lhs, rhs)
}

/// Pointwise multiplication is associative on a quinary product of function spaces.
theorem function_quintuple_mul_assoc[T, A: Semigroup, B: Semigroup, C: Semigroup, D: Semigroup, E: Semigroup](
    p: Pair[T -> A, Pair[T -> B, Pair[T -> C, Pair[T -> D, T -> E]]]],
    q: Pair[T -> A, Pair[T -> B, Pair[T -> C, Pair[T -> D, T -> E]]]],
    r: Pair[T -> A, Pair[T -> B, Pair[T -> C, Pair[T -> D, T -> E]]]]
) {
    function_quintuple_mul(p, function_quintuple_mul(q, r)) =
    function_quintuple_mul(function_quintuple_mul(p, q), r)
} by {
    let lhs = function_quintuple_mul(p, function_quintuple_mul(q, r))
    let rhs = function_quintuple_mul(function_quintuple_mul(p, q), r)
    lhs.first = pointwise_mul(p.first, function_quintuple_mul(q, r).first)
    function_quintuple_mul(q, r).first = pointwise_mul(q.first, r.first)
    lhs.first = pointwise_mul(p.first, pointwise_mul(q.first, r.first))
    rhs.first = pointwise_mul(function_quintuple_mul(p, q).first, r.first)
    function_quintuple_mul(p, q).first = pointwise_mul(p.first, q.first)
    rhs.first = pointwise_mul(pointwise_mul(p.first, q.first), r.first)
    pointwise_mul_assoc(p.first, q.first, r.first)
    lhs.first = rhs.first
    lhs.second = function_quadruple_mul(p.second, function_quintuple_mul(q, r).second)
    function_quintuple_mul(q, r).second = function_quadruple_mul(q.second, r.second)
    lhs.second = function_quadruple_mul(p.second, function_quadruple_mul(q.second, r.second))
    rhs.second = function_quadruple_mul(function_quintuple_mul(p, q).second, r.second)
    function_quintuple_mul(p, q).second = function_quadruple_mul(p.second, q.second)
    rhs.second = function_quadruple_mul(function_quadruple_mul(p.second, q.second), r.second)
    function_quadruple_mul_assoc(p.second, q.second, r.second)
    lhs.second = rhs.second
    pair_ext(lhs, rhs)
}

/// Pointwise multiplication is commutative on a quinary product of function spaces.
theorem function_quintuple_mul_comm[T, A: CommSemigroup, B: CommSemigroup, C: CommSemigroup, D: CommSemigroup, E: CommSemigroup](
    p: Pair[T -> A, Pair[T -> B, Pair[T -> C, Pair[T -> D, T -> E]]]],
    q: Pair[T -> A, Pair[T -> B, Pair[T -> C, Pair[T -> D, T -> E]]]]
) {
    function_quintuple_mul(p, q) = function_quintuple_mul(q, p)
} by {
    let lhs = function_quintuple_mul(p, q)
    let rhs = function_quintuple_mul(q, p)
    lhs.first = pointwise_mul(p.first, q.first)
    rhs.first = pointwise_mul(q.first, p.first)
    pointwise_mul_comm(p.first, q.first)
    lhs.first = rhs.first
    lhs.second = function_quadruple_mul(p.second, q.second)
    rhs.second = function_quadruple_mul(q.second, p.second)
    function_quadruple_mul_comm(p.second, q.second)
    lhs.second = rhs.second
    pair_ext(lhs, rhs)
}

/// Multiplying by the quinary pointwise one on the right changes no coordinate.
theorem function_quintuple_mul_one_right[T, A: Monoid, B: Monoid, C: Monoid, D: Monoid, E: Monoid](
    p: Pair[T -> A, Pair[T -> B, Pair[T -> C, Pair[T -> D, T -> E]]]]
) {
    function_quintuple_mul(p, function_quintuple_one[T, A, B, C, D, E]) = p
} by {
    let lhs = function_quintuple_mul(p, function_quintuple_one[T, A, B, C, D, E])
    lhs.first = pointwise_mul(p.first, function_quintuple_one[T, A, B, C, D, E].first)
    function_quintuple_one[T, A, B, C, D, E].first = pointwise_one[T, A]
    lhs.first = pointwise_mul(p.first, pointwise_one[T, A])
    pointwise_mul_one_right(p.first)
    lhs.first = p.first
    lhs.second = function_quadruple_mul(p.second, function_quintuple_one[T, A, B, C, D, E].second)
    function_quintuple_one[T, A, B, C, D, E].second = function_quadruple_one[T, B, C, D, E]
    lhs.second = function_quadruple_mul(p.second, function_quadruple_one[T, B, C, D, E])
    function_quadruple_mul_one_right(p.second)
    lhs.second = p.second
    pair_ext(lhs, p)
}

/// Multiplying by the quinary pointwise one on the left changes no coordinate.
theorem function_quintuple_mul_one_left[T, A: Monoid, B: Monoid, C: Monoid, D: Monoid, E: Monoid](
    p: Pair[T -> A, Pair[T -> B, Pair[T -> C, Pair[T -> D, T -> E]]]]
) {
    function_quintuple_mul(function_quintuple_one[T, A, B, C, D, E], p) = p
} by {
    let lhs = function_quintuple_mul(function_quintuple_one[T, A, B, C, D, E], p)
    lhs.first = pointwise_mul(function_quintuple_one[T, A, B, C, D, E].first, p.first)
    function_quintuple_one[T, A, B, C, D, E].first = pointwise_one[T, A]
    lhs.first = pointwise_mul(pointwise_one[T, A], p.first)
    pointwise_mul_one_left(p.first)
    lhs.first = p.first
    lhs.second = function_quadruple_mul(function_quintuple_one[T, A, B, C, D, E].second, p.second)
    function_quintuple_one[T, A, B, C, D, E].second = function_quadruple_one[T, B, C, D, E]
    lhs.second = function_quadruple_mul(function_quadruple_one[T, B, C, D, E], p.second)
    function_quadruple_mul_one_left(p.second)
    lhs.second = p.second
    pair_ext(lhs, p)
}

/// Pointwise inversion is a right inverse on a quinary product of function spaces.
theorem function_quintuple_mul_inverse_right[T, A: Group, B: Group, C: Group, D: Group, E: Group](
    p: Pair[T -> A, Pair[T -> B, Pair[T -> C, Pair[T -> D, T -> E]]]]
) {
    function_quintuple_mul(p, function_quintuple_inverse(p)) =
    function_quintuple_one[T, A, B, C, D, E]
} by {
    let lhs = function_quintuple_mul(p, function_quintuple_inverse(p))
    let rhs = function_quintuple_one[T, A, B, C, D, E]
    lhs.first = pointwise_mul(p.first, function_quintuple_inverse(p).first)
    function_quintuple_inverse(p).first = pointwise_inverse(p.first)
    lhs.first = pointwise_mul(p.first, pointwise_inverse(p.first))
    pointwise_mul_inverse_right(p.first)
    lhs.first = pointwise_one[T, A]
    rhs.first = pointwise_one[T, A]
    lhs.second = function_quadruple_mul(p.second, function_quintuple_inverse(p).second)
    function_quintuple_inverse(p).second = function_quadruple_inverse(p.second)
    lhs.second = function_quadruple_mul(p.second, function_quadruple_inverse(p.second))
    function_quadruple_mul_inverse_right(p.second)
    lhs.second = function_quadruple_one[T, B, C, D, E]
    rhs.second = function_quadruple_one[T, B, C, D, E]
    pair_ext(lhs, rhs)
}

/// Pointwise inversion is a left inverse on a quinary product of function spaces.
theorem function_quintuple_mul_inverse_left[T, A: Group, B: Group, C: Group, D: Group, E: Group](
    p: Pair[T -> A, Pair[T -> B, Pair[T -> C, Pair[T -> D, T -> E]]]]
) {
    function_quintuple_mul(function_quintuple_inverse(p), p) =
    function_quintuple_one[T, A, B, C, D, E]
} by {
    let lhs = function_quintuple_mul(function_quintuple_inverse(p), p)
    let rhs = function_quintuple_one[T, A, B, C, D, E]
    lhs.first = pointwise_mul(function_quintuple_inverse(p).first, p.first)
    function_quintuple_inverse(p).first = pointwise_inverse(p.first)
    lhs.first = pointwise_mul(pointwise_inverse(p.first), p.first)
    pointwise_mul_inverse_left(p.first)
    lhs.first = pointwise_one[T, A]
    rhs.first = pointwise_one[T, A]
    lhs.second = function_quadruple_mul(function_quintuple_inverse(p).second, p.second)
    function_quintuple_inverse(p).second = function_quadruple_inverse(p.second)
    lhs.second = function_quadruple_mul(function_quadruple_inverse(p.second), p.second)
    function_quadruple_mul_inverse_left(p.second)
    lhs.second = function_quadruple_one[T, B, C, D, E]
    rhs.second = function_quadruple_one[T, B, C, D, E]
    pair_ext(lhs, rhs)
}

/// Pointwise multiplication distributes over quinary pointwise addition on the left.
theorem function_quintuple_mul_distrib_left[T, A: Semiring, B: Semiring, C: Semiring, D: Semiring, E: Semiring](
    p: Pair[T -> A, Pair[T -> B, Pair[T -> C, Pair[T -> D, T -> E]]]],
    q: Pair[T -> A, Pair[T -> B, Pair[T -> C, Pair[T -> D, T -> E]]]],
    r: Pair[T -> A, Pair[T -> B, Pair[T -> C, Pair[T -> D, T -> E]]]]
) {
    function_quintuple_mul(p, function_quintuple_add(q, r)) =
    function_quintuple_add(function_quintuple_mul(p, q), function_quintuple_mul(p, r))
} by {
    let lhs = function_quintuple_mul(p, function_quintuple_add(q, r))
    let rhs = function_quintuple_add(function_quintuple_mul(p, q), function_quintuple_mul(p, r))
    lhs.first = pointwise_mul(p.first, function_quintuple_add(q, r).first)
    function_quintuple_add(q, r).first = pointwise_add(q.first, r.first)
    lhs.first = pointwise_mul(p.first, pointwise_add(q.first, r.first))
    rhs.first = pointwise_add(function_quintuple_mul(p, q).first, function_quintuple_mul(p, r).first)
    function_quintuple_mul(p, q).first = pointwise_mul(p.first, q.first)
    function_quintuple_mul(p, r).first = pointwise_mul(p.first, r.first)
    rhs.first = pointwise_add(pointwise_mul(p.first, q.first), pointwise_mul(p.first, r.first))
    pointwise_mul_distrib_left(p.first, q.first, r.first)
    lhs.first = rhs.first
    lhs.second = function_quadruple_mul(p.second, function_quintuple_add(q, r).second)
    function_quintuple_add(q, r).second = function_quadruple_add(q.second, r.second)
    lhs.second = function_quadruple_mul(p.second, function_quadruple_add(q.second, r.second))
    rhs.second = function_quadruple_add(function_quintuple_mul(p, q).second, function_quintuple_mul(p, r).second)
    function_quintuple_mul(p, q).second = function_quadruple_mul(p.second, q.second)
    function_quintuple_mul(p, r).second = function_quadruple_mul(p.second, r.second)
    rhs.second = function_quadruple_add(function_quadruple_mul(p.second, q.second),
        function_quadruple_mul(p.second, r.second))
    function_quadruple_mul_distrib_left(p.second, q.second, r.second)
    lhs.second = rhs.second
    pair_ext(lhs, rhs)
}

/// Pointwise multiplication distributes over quinary pointwise addition on the right.
theorem function_quintuple_mul_distrib_right[T, A: Semiring, B: Semiring, C: Semiring, D: Semiring, E: Semiring](
    p: Pair[T -> A, Pair[T -> B, Pair[T -> C, Pair[T -> D, T -> E]]]],
    q: Pair[T -> A, Pair[T -> B, Pair[T -> C, Pair[T -> D, T -> E]]]],
    r: Pair[T -> A, Pair[T -> B, Pair[T -> C, Pair[T -> D, T -> E]]]]
) {
    function_quintuple_mul(function_quintuple_add(p, q), r) =
    function_quintuple_add(function_quintuple_mul(p, r), function_quintuple_mul(q, r))
} by {
    let lhs = function_quintuple_mul(function_quintuple_add(p, q), r)
    let rhs = function_quintuple_add(function_quintuple_mul(p, r), function_quintuple_mul(q, r))
    lhs.first = pointwise_mul(function_quintuple_add(p, q).first, r.first)
    function_quintuple_add(p, q).first = pointwise_add(p.first, q.first)
    lhs.first = pointwise_mul(pointwise_add(p.first, q.first), r.first)
    rhs.first = pointwise_add(function_quintuple_mul(p, r).first, function_quintuple_mul(q, r).first)
    function_quintuple_mul(p, r).first = pointwise_mul(p.first, r.first)
    function_quintuple_mul(q, r).first = pointwise_mul(q.first, r.first)
    rhs.first = pointwise_add(pointwise_mul(p.first, r.first), pointwise_mul(q.first, r.first))
    pointwise_mul_distrib_right(p.first, q.first, r.first)
    lhs.first = rhs.first
    lhs.second = function_quadruple_mul(function_quintuple_add(p, q).second, r.second)
    function_quintuple_add(p, q).second = function_quadruple_add(p.second, q.second)
    lhs.second = function_quadruple_mul(function_quadruple_add(p.second, q.second), r.second)
    rhs.second = function_quadruple_add(function_quintuple_mul(p, r).second, function_quintuple_mul(q, r).second)
    function_quintuple_mul(p, r).second = function_quadruple_mul(p.second, r.second)
    function_quintuple_mul(q, r).second = function_quadruple_mul(q.second, r.second)
    rhs.second = function_quadruple_add(function_quadruple_mul(p.second, r.second),
        function_quadruple_mul(q.second, r.second))
    function_quadruple_mul_distrib_right(p.second, q.second, r.second)
    lhs.second = rhs.second
    pair_ext(lhs, rhs)
}

/// Multiplying by the quinary pointwise zero on the right gives the pointwise zero.
theorem function_quintuple_mul_zero_right[T, A: Semiring, B: Semiring, C: Semiring, D: Semiring, E: Semiring](
    p: Pair[T -> A, Pair[T -> B, Pair[T -> C, Pair[T -> D, T -> E]]]]
) {
    function_quintuple_mul(p, function_quintuple_zero[T, A, B, C, D, E]) =
    function_quintuple_zero[T, A, B, C, D, E]
} by {
    let lhs = function_quintuple_mul(p, function_quintuple_zero[T, A, B, C, D, E])
    let rhs = function_quintuple_zero[T, A, B, C, D, E]
    lhs.first = pointwise_mul(p.first, function_quintuple_zero[T, A, B, C, D, E].first)
    function_quintuple_zero[T, A, B, C, D, E].first = pointwise_zero[T, A]
    lhs.first = pointwise_mul(p.first, pointwise_zero[T, A])
    pointwise_mul_zero_right(p.first)
    lhs.first = pointwise_zero[T, A]
    rhs.first = pointwise_zero[T, A]
    lhs.second = function_quadruple_mul(p.second, function_quintuple_zero[T, A, B, C, D, E].second)
    function_quintuple_zero[T, A, B, C, D, E].second = function_quadruple_zero[T, B, C, D, E]
    lhs.second = function_quadruple_mul(p.second, function_quadruple_zero[T, B, C, D, E])
    function_quadruple_mul_zero_right(p.second)
    lhs.second = function_quadruple_zero[T, B, C, D, E]
    rhs.second = function_quadruple_zero[T, B, C, D, E]
    pair_ext(lhs, rhs)
}

/// Multiplying by the quinary pointwise zero on the left gives the pointwise zero.
theorem function_quintuple_mul_zero_left[T, A: Semiring, B: Semiring, C: Semiring, D: Semiring, E: Semiring](
    p: Pair[T -> A, Pair[T -> B, Pair[T -> C, Pair[T -> D, T -> E]]]]
) {
    function_quintuple_mul(function_quintuple_zero[T, A, B, C, D, E], p) =
    function_quintuple_zero[T, A, B, C, D, E]
} by {
    let lhs = function_quintuple_mul(function_quintuple_zero[T, A, B, C, D, E], p)
    let rhs = function_quintuple_zero[T, A, B, C, D, E]
    lhs.first = pointwise_mul(function_quintuple_zero[T, A, B, C, D, E].first, p.first)
    function_quintuple_zero[T, A, B, C, D, E].first = pointwise_zero[T, A]
    lhs.first = pointwise_mul(pointwise_zero[T, A], p.first)
    pointwise_mul_zero_left(p.first)
    lhs.first = pointwise_zero[T, A]
    rhs.first = pointwise_zero[T, A]
    lhs.second = function_quadruple_mul(function_quintuple_zero[T, A, B, C, D, E].second, p.second)
    function_quintuple_zero[T, A, B, C, D, E].second = function_quadruple_zero[T, B, C, D, E]
    lhs.second = function_quadruple_mul(function_quadruple_zero[T, B, C, D, E], p.second)
    function_quadruple_mul_zero_left(p.second)
    lhs.second = function_quadruple_zero[T, B, C, D, E]
    rhs.second = function_quadruple_zero[T, B, C, D, E]
    pair_ext(lhs, rhs)
}
