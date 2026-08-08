from int import Int
from data.basic.functions import is_injective_fn
from rat import Rat, from_int_cancel, from_int_num, from_int_denom, from_int_zero, reduce, reduce_idempotent
from algebra.add_monoid import is_add_monoid_hom
from algebra.monoid.monoid import is_monoid_hom
from semiring import Semiring
from algebra.semiring_hom import SemiringHom, is_semiring_hom

/// The embedding of the integers in the rationals preserves addition.
///
/// Both sides reduce the same fraction: the denominators are one, so the general addition
/// rule collapses to adding the numerators.
theorem from_int_add(a: Int, b: Int) {
    Rat.from_int(a) + Rat.from_int(b) = Rat.from_int(a + b)
} by {
    from_int_num(a)
    from_int_denom(a)
    from_int_num(b)
    from_int_denom(b)
    (Rat.from_int(a) + Rat.from_int(b) = reduce(
        Rat.from_int(a).num * Rat.from_int(b).denom
            + Rat.from_int(b).num * Rat.from_int(a).denom,
        Rat.from_int(a).denom * Rat.from_int(b).denom))
    a * Int.1 = a
    b * Int.1 = b
    Int.1 * Int.1 = Int.1
    Rat.from_int(a) + Rat.from_int(b) = reduce(a + b, Int.1)
    from_int_num(a + b)
    from_int_denom(a + b)
    reduce_idempotent(Rat.from_int(a + b))
    reduce(Rat.from_int(a + b).num, Rat.from_int(a + b).denom) = Rat.from_int(a + b)
    reduce(a + b, Int.1) = Rat.from_int(a + b)
    Rat.from_int(a) + Rat.from_int(b) = Rat.from_int(a + b)
}

/// The embedding of the integers in the rationals preserves multiplication.
theorem from_int_mul(a: Int, b: Int) {
    Rat.from_int(a) * Rat.from_int(b) = Rat.from_int(a * b)
} by {
    from_int_num(a)
    from_int_denom(a)
    from_int_num(b)
    from_int_denom(b)
    (Rat.from_int(a) * Rat.from_int(b) = reduce(
        Rat.from_int(a).num * Rat.from_int(b).num,
        Rat.from_int(a).denom * Rat.from_int(b).denom))
    Int.1 * Int.1 = Int.1
    Rat.from_int(a) * Rat.from_int(b) = reduce(a * b, Int.1)
    from_int_num(a * b)
    from_int_denom(a * b)
    reduce_idempotent(Rat.from_int(a * b))
    reduce(Rat.from_int(a * b).num, Rat.from_int(a * b).denom) = Rat.from_int(a * b)
    reduce(a * b, Int.1) = Rat.from_int(a * b)
    Rat.from_int(a) * Rat.from_int(b) = Rat.from_int(a * b)
}

/// The embedding takes one to one.
///
/// True by construction: `Rat.1` is defined as the image of `Int.1`.
theorem from_int_one {
    Rat.from_int(Int.1) = Rat.1
}

/// The embedding is an additive monoid homomorphism.
theorem from_int_is_add_monoid_hom {
    is_add_monoid_hom(Rat.from_int)
} by {
    from_int_zero
    Rat.from_int(Int.0) = Rat.0
    forall(a: Int, b: Int) {
        from_int_add(a, b)
        Rat.from_int(a + b) = Rat.from_int(a) + Rat.from_int(b)
    }
    is_add_monoid_hom(Rat.from_int) = (Rat.from_int(Int.0) = Rat.0 and forall(a: Int, b: Int) {
        Rat.from_int(a + b) = Rat.from_int(a) + Rat.from_int(b)
    })
    is_add_monoid_hom(Rat.from_int)
}

/// The embedding is a multiplicative monoid homomorphism.
theorem from_int_is_monoid_hom {
    is_monoid_hom(Rat.from_int)
} by {
    from_int_one
    Rat.from_int(Int.1) = Rat.1
    forall(a: Int, b: Int) {
        from_int_mul(a, b)
        Rat.from_int(a * b) = Rat.from_int(a) * Rat.from_int(b)
    }
    is_monoid_hom(Rat.from_int) = (Rat.from_int(Int.1) = Rat.1 and forall(a: Int, b: Int) {
        Rat.from_int(a * b) = Rat.from_int(a) * Rat.from_int(b)
    })
    is_monoid_hom(Rat.from_int)
}

/// The embedding of the integers in the rationals is a semiring homomorphism.
///
/// `src/rat/interface.ac` had `from_int_zero`, `from_int_num`, `from_int_denom`, and
/// `from_int_cancel` for injectivity, but nothing saying the embedding respects the ring
/// operations. Without that it could not be packaged as a homomorphism, which is what
/// transporting a statement about integer polynomials into the rationals needs.
theorem from_int_is_semiring_hom {
    is_semiring_hom(Rat.from_int)
} by {
    from_int_is_add_monoid_hom
    is_add_monoid_hom(Rat.from_int)
    from_int_is_monoid_hom
    is_monoid_hom(Rat.from_int)
    is_semiring_hom(Rat.from_int) =
        (is_add_monoid_hom(Rat.from_int) and is_monoid_hom(Rat.from_int))
    is_semiring_hom(Rat.from_int)
}

/// The embedding of the integers in the rationals, as a homomorphism.
///
/// The packaged form, which is what `polynomial_map` takes.
let int_to_rat_hom: SemiringHom[Int, Rat] satisfy {
    SemiringHom.new(Rat.from_int) = Option.some(int_to_rat_hom)
}

/// The packaged homomorphism acts by the embedding.
theorem int_to_rat_hom_hom {
    int_to_rat_hom.hom = Rat.from_int
} by {
}

/// The packaged homomorphism agrees with the embedding at every integer.
theorem int_to_rat_hom_apply(a: Int) {
    int_to_rat_hom.hom(a) = Rat.from_int(a)
} by {
    int_to_rat_hom_hom
    int_to_rat_hom.hom = Rat.from_int
}

/// The embedding of the integers in the rationals is injective.
///
/// `from_int_cancel` states this as a cancellation; this is the packaged form the list and
/// polynomial machinery expects.
theorem int_to_rat_is_injective {
    is_injective_fn(Rat.from_int)
} by {
    forall(a: Int, b: Int) {
        if Rat.from_int(a) = Rat.from_int(b) {
            from_int_cancel(a, b)
            a = b
        }
        (Rat.from_int(a) = Rat.from_int(b) implies a = b)
    }
    is_injective_fn(Rat.from_int) = forall(a: Int, b: Int) {
        Rat.from_int(a) = Rat.from_int(b) implies a = b
    }
    is_injective_fn(Rat.from_int)
}

/// An integer vanishes exactly when its image in the rationals does.
theorem int_zero_iff_rat_zero(a: Int) {
    (a = Int.0) = (Rat.from_int(a) = Rat.0)
} by {
    if a = Int.0 {
        from_int_zero
        Rat.from_int(Int.0) = Rat.0
        Rat.from_int(a) = Rat.0
    }
    if Rat.from_int(a) = Rat.0 {
        from_int_zero
        Rat.from_int(Int.0) = Rat.0
        Rat.from_int(a) = Rat.from_int(Int.0)
        from_int_cancel(a, Int.0)
        a = Int.0
    }
    ((a = Int.0) implies (Rat.from_int(a) = Rat.0))
    ((Rat.from_int(a) = Rat.0) implies (a = Int.0))
    (a = Int.0) = (Rat.from_int(a) = Rat.0)
}
