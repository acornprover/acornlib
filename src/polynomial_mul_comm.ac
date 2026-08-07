from nat import Nat, add_sub, add_imp_sub_left
from comm_ring import CommRing
from data.basic.functions import function_extensionality
from list import partial
from polynomial import Polynomial, polynomial_mul, polynomial_mul_coeff,
    polynomial_mul_term_coeff, polynomial_mul_coeff_apply, polynomial_ext_pointwise,
    polynomial_one_coeff_zero, polynomial_one_coeff_of_ne_zero
from data.nat.nat_range_sum import range_sum, range_sum_congr
from data.nat.nat_range_sum_bridge import range_sum_eq_partial
from data.nat.nat_range_sum_reverse import rev_fn, range_sum_rev
from data.nat.nat_residue_range_sum import point_fn, point_fn_at, point_fn_off, range_sum_point_fn

/// The convolution summand read backwards is the summand of the reversed product.
///
/// At index `i` the reversed summand reads position `k - i`, which pairs the coefficient of `p`
/// there with the coefficient of `q` at `i`. That is the summand of the product taken the other
/// way round, once the ring is commutative.
theorem mul_term_coeff_rev[R: CommRing](
    p: Polynomial[R], q: Polynomial[R], k: Nat, i: Nat
) {
    i <= k implies rev_fn(polynomial_mul_term_coeff(p, q, k), k.suc, i)
        = polynomial_mul_term_coeff(q, p, k, i)
} by {
    if i <= k {
        (rev_fn(polynomial_mul_term_coeff(p, q, k), k.suc, i)
            = polynomial_mul_term_coeff(p, q, k, k.suc - i.suc))
        add_sub(k, i)
        ((k - i) + i = k)
        ((k - i) + i.suc = k.suc)
        add_imp_sub_left(i.suc, k - i, k.suc)
        (k.suc - i.suc = k - i)
        (polynomial_mul_term_coeff(p, q, k, k - i)
            = p.coeff(k - i) * q.coeff(k - (k - i)))
        add_imp_sub_left(k - i, i, k)
        (k - (k - i) = i)
        (rev_fn(polynomial_mul_term_coeff(p, q, k), k.suc, i)
            = p.coeff(k - i) * q.coeff(i))
        (polynomial_mul_term_coeff(q, p, k, i) = q.coeff(i) * p.coeff(k - i))
        (p.coeff(k - i) * q.coeff(i) = q.coeff(i) * p.coeff(k - i))
        (rev_fn(polynomial_mul_term_coeff(p, q, k), k.suc, i)
            = polynomial_mul_term_coeff(q, p, k, i))
    }
}

/// The convolution coefficient function is the partially applied summand.
///
/// The definition writes an inline lambda; every statement here works with the partial
/// application instead, and these are the same function.
theorem mul_coeff_eq_range_sum[R: CommRing](
    p: Polynomial[R], q: Polynomial[R], k: Nat
) {
    (polynomial_mul_coeff(p, q, k)
        = range_sum(polynomial_mul_term_coeff(p, q, k), k.suc))
} by {
    forall(i: Nat) {
        ((function(j: Nat) { polynomial_mul_term_coeff(p, q, k, j) })(i)
            = polynomial_mul_term_coeff(p, q, k)(i))
    }
    function_extensionality[Nat, R](
        function(j: Nat) { polynomial_mul_term_coeff(p, q, k, j) },
        polynomial_mul_term_coeff(p, q, k))
    ((function(j: Nat) { polynomial_mul_term_coeff(p, q, k, j) })
        = polynomial_mul_term_coeff(p, q, k))
    (polynomial_mul_coeff(p, q, k)
        = partial(function(j: Nat) { polynomial_mul_term_coeff(p, q, k, j) }, k.suc))
    (polynomial_mul_coeff(p, q, k)
        = partial(polynomial_mul_term_coeff(p, q, k), k.suc))
    range_sum_eq_partial(polynomial_mul_term_coeff(p, q, k), k.suc)
    (range_sum(polynomial_mul_term_coeff(p, q, k), k.suc)
        = partial(polynomial_mul_term_coeff(p, q, k), k.suc))
    (polynomial_mul_coeff(p, q, k)
        = range_sum(polynomial_mul_term_coeff(p, q, k), k.suc))
}

/// The convolution coefficient does not depend on the order of the factors.
theorem polynomial_mul_coeff_comm[R: CommRing](
    p: Polynomial[R], q: Polynomial[R], k: Nat
) {
    polynomial_mul_coeff(p, q, k) = polynomial_mul_coeff(q, p, k)
} by {
    mul_coeff_eq_range_sum(p, q, k)
    (polynomial_mul_coeff(p, q, k)
        = range_sum(polynomial_mul_term_coeff(p, q, k), k.suc))
    range_sum_rev(polynomial_mul_term_coeff(p, q, k), k.suc)
    (range_sum(polynomial_mul_term_coeff(p, q, k), k.suc)
        = range_sum(rev_fn(polynomial_mul_term_coeff(p, q, k), k.suc), k.suc))
    forall(i: Nat) {
        if i < k.suc {
            i <= k
            mul_term_coeff_rev(p, q, k, i)
            (rev_fn(polynomial_mul_term_coeff(p, q, k), k.suc, i)
                = polynomial_mul_term_coeff(q, p, k, i))
        }
        (i < k.suc implies rev_fn(polynomial_mul_term_coeff(p, q, k), k.suc)(i)
            = polynomial_mul_term_coeff(q, p, k)(i))
    }
    range_sum_congr(rev_fn(polynomial_mul_term_coeff(p, q, k), k.suc),
        polynomial_mul_term_coeff(q, p, k), k.suc)
    (range_sum(rev_fn(polynomial_mul_term_coeff(p, q, k), k.suc), k.suc)
        = range_sum(polynomial_mul_term_coeff(q, p, k), k.suc))
    mul_coeff_eq_range_sum(q, p, k)
    (polynomial_mul_coeff(q, p, k)
        = range_sum(polynomial_mul_term_coeff(q, p, k), k.suc))
    polynomial_mul_coeff(p, q, k) = polynomial_mul_coeff(q, p, k)
}

/// Polynomial multiplication over a commutative ring is commutative.
///
/// One of the ring axioms `Polynomial[R]` would need to carry a `Mul` instance. The convolution
/// is symmetric because reversing the summation index exchanges the two factors, which is the
/// reindexing identity and nothing more.
theorem polynomial_mul_comm[R: CommRing](p: Polynomial[R], q: Polynomial[R]) {
    polynomial_mul(p, q) = polynomial_mul(q, p)
} by {
    forall(k: Nat) {
        polynomial_mul_coeff_apply(p, q, k)
        (polynomial_mul(p, q).coeff(k) = polynomial_mul_coeff(p, q, k))
        polynomial_mul_coeff_apply(q, p, k)
        (polynomial_mul(q, p).coeff(k) = polynomial_mul_coeff(q, p, k))
        polynomial_mul_coeff_comm(p, q, k)
        (polynomial_mul_coeff(p, q, k) = polynomial_mul_coeff(q, p, k))
        polynomial_mul(p, q).coeff(k) = polynomial_mul(q, p).coeff(k)
    }
    polynomial_ext_pointwise(polynomial_mul(p, q), polynomial_mul(q, p))
    polynomial_mul(p, q) = polynomial_mul(q, p)
}

/// The convolution against one is the coefficient itself.
///
/// Only the top summand survives: below it the second factor is read away from zero, where the
/// one polynomial vanishes. So the coefficient function of the product is a point summand and
/// sums to a single term.
theorem polynomial_mul_coeff_one[R: CommRing](p: Polynomial[R], k: Nat) {
    polynomial_mul_coeff(p, Polynomial[R].one, k) = p.coeff(k)
} by {
    mul_coeff_eq_range_sum(p, Polynomial[R].one, k)
    (polynomial_mul_coeff(p, Polynomial[R].one, k)
        = range_sum(polynomial_mul_term_coeff(p, Polynomial[R].one, k), k.suc))
    forall(i: Nat) {
        if i < k.suc {
            i <= k
            (polynomial_mul_term_coeff(p, Polynomial[R].one, k, i)
                = p.coeff(i) * Polynomial[R].one.coeff(k - i))
            if i = k {
                (k - i = Nat.0)
                (Polynomial[R].one.coeff(Nat.0) = R.1)
                (p.coeff(i) * R.1 = p.coeff(i))
                (polynomial_mul_term_coeff(p, Polynomial[R].one, k, i) = p.coeff(k))
                point_fn_at(k, p.coeff(k))
                (point_fn(k, p.coeff(k))(i) = p.coeff(k))
                (polynomial_mul_term_coeff(p, Polynomial[R].one, k)(i)
                    = point_fn(k, p.coeff(k))(i))
            }
            if i != k {
                i < k
                add_sub(k, i)
                ((k - i) + i = k)
                if k - i = Nat.0 {
                    (Nat.0 + i = k)
                    i = k
                    false
                }
                k - i != Nat.0
                polynomial_one_coeff_of_ne_zero[R](k - i)
                (Polynomial[R].one.coeff(k - i) = R.0)
                (p.coeff(i) * R.0 = R.0)
                (polynomial_mul_term_coeff(p, Polynomial[R].one, k, i) = R.0)
                point_fn_off(k, p.coeff(k), i)
                (point_fn(k, p.coeff(k))(i) = R.0)
                (polynomial_mul_term_coeff(p, Polynomial[R].one, k)(i)
                    = point_fn(k, p.coeff(k))(i))
            }
            (polynomial_mul_term_coeff(p, Polynomial[R].one, k)(i)
                = point_fn(k, p.coeff(k))(i))
        }
        (i < k.suc implies polynomial_mul_term_coeff(p, Polynomial[R].one, k)(i)
            = point_fn(k, p.coeff(k))(i))
    }
    range_sum_congr(polynomial_mul_term_coeff(p, Polynomial[R].one, k),
        point_fn(k, p.coeff(k)), k.suc)
    (range_sum(polynomial_mul_term_coeff(p, Polynomial[R].one, k), k.suc)
        = range_sum(point_fn(k, p.coeff(k)), k.suc))
    k < k.suc
    range_sum_point_fn(k, p.coeff(k), k.suc)
    (range_sum(point_fn(k, p.coeff(k)), k.suc) = p.coeff(k))
    polynomial_mul_coeff(p, Polynomial[R].one, k) = p.coeff(k)
}

/// One is a right identity for polynomial multiplication.
theorem polynomial_mul_one_right[R: CommRing](p: Polynomial[R]) {
    polynomial_mul(p, Polynomial[R].one) = p
} by {
    forall(k: Nat) {
        polynomial_mul_coeff_apply(p, Polynomial[R].one, k)
        (polynomial_mul(p, Polynomial[R].one).coeff(k)
            = polynomial_mul_coeff(p, Polynomial[R].one, k))
        polynomial_mul_coeff_one(p, k)
        (polynomial_mul_coeff(p, Polynomial[R].one, k) = p.coeff(k))
        polynomial_mul(p, Polynomial[R].one).coeff(k) = p.coeff(k)
    }
    polynomial_ext_pointwise(polynomial_mul(p, Polynomial[R].one), p)
    polynomial_mul(p, Polynomial[R].one) = p
}

/// One is a left identity for polynomial multiplication.
theorem polynomial_mul_one_left[R: CommRing](p: Polynomial[R]) {
    polynomial_mul(Polynomial[R].one, p) = p
} by {
    polynomial_mul_comm(Polynomial[R].one, p)
    (polynomial_mul(Polynomial[R].one, p) = polynomial_mul(p, Polynomial[R].one))
    polynomial_mul_one_right(p)
    polynomial_mul(Polynomial[R].one, p) = p
}
