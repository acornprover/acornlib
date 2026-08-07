from order import PartialOrder, LinearOrder, lte_refl, gte_refl, lte_trans, gte_trans, lte_antisymm, lt_trans, gt_trans, not_lt_ref, not_gt_ref, lte_or_gte
from order_cases import lt_or_eq_of_not_gt, gt_or_eq_of_not_lt
from data.basic.relation import relation_refl_trans_closure, relation_trans_closure,
    relation_subset_refl_trans_closure, relation_subset_trans_closure,
    relation_refl_trans_closure_eq_eq_or_trans_closure,
    relation_refl_trans_closure_subset_of_reflexive_transitive,
    relation_trans_closure_subset_of_transitive, relation_refl_trans_closure_is_reflexive,
    relation_refl_trans_closure_pullback_of_bijection, relation_trans_closure_pullback_of_surjective
from data.basic.functions import binary_function_extensionality, binary_function_eq_transport_predicate,
    binary_function_eq_transport_predicate_rev, is_injective_fn, is_bijection_fn, is_surjective_fn,
    Bijection, bijection_map_is_injective, bijection_map_is_bijection
from data.basic.relation_basic import is_reflexive, is_transitive, is_antisymmetric, is_irreflexive,
    is_asymmetric, is_total, reflexive_self, relation_converse, relation_subset,
    relation_subset_refl, relation_subset_step, relation_converse_is_reflexive,
    relation_converse_is_transitive, relation_converse_is_total,
    relation_converse_is_antisymmetric
from data.basic.relation_transport import relation_pullback, relation_pullback_is_reflexive,
    relation_pullback_is_irreflexive, relation_pullback_is_asymmetric,
    relation_pullback_is_transitive, relation_pullback_is_total,
    relation_pullback_is_antisymmetric_of_injective

/// True if a relation has the data of a partial order.
define is_partial_order_relation[T](r: (T, T) -> Bool) -> Bool {
    is_reflexive(r) and is_transitive(r) and is_antisymmetric(r)
}

/// True if a relation has the data of a linear order.
define is_linear_order_relation[T](r: (T, T) -> Bool) -> Bool {
    is_partial_order_relation(r) and is_total(r)
}

/// The strict part of a relation.
define relation_strict_part[T](r: (T, T) -> Bool, a: T, b: T) -> Bool {
    r(a, b) and a != b
}

theorem partial_order_relation_is_reflexive[T](r: (T, T) -> Bool) {
    is_partial_order_relation(r) implies is_reflexive(r)
} by {
    if is_partial_order_relation(r) {
        is_partial_order_relation(r) = (is_reflexive(r) and is_transitive(r) and is_antisymmetric(r))
        is_reflexive(r)
    }
}

theorem partial_order_relation_is_transitive[T](r: (T, T) -> Bool) {
    is_partial_order_relation(r) implies is_transitive(r)
} by {
    if is_partial_order_relation(r) {
        is_partial_order_relation(r) = (is_reflexive(r) and is_transitive(r) and is_antisymmetric(r))
        is_transitive(r)
    }
}

theorem partial_order_relation_is_antisymmetric[T](r: (T, T) -> Bool) {
    is_partial_order_relation(r) implies is_antisymmetric(r)
} by {
    if is_partial_order_relation(r) {
        is_partial_order_relation(r) = (is_reflexive(r) and is_transitive(r) and is_antisymmetric(r))
        is_antisymmetric(r)
    }
}

theorem linear_order_relation_is_partial_order[T](r: (T, T) -> Bool) {
    is_linear_order_relation(r) implies is_partial_order_relation(r)
} by {
    if is_linear_order_relation(r) {
        is_linear_order_relation(r) = (is_partial_order_relation(r) and is_total(r))
        is_partial_order_relation(r)
    }
}

theorem linear_order_relation_is_total[T](r: (T, T) -> Bool) {
    is_linear_order_relation(r) implies is_total(r)
} by {
    if is_linear_order_relation(r) {
        is_linear_order_relation(r) = (is_partial_order_relation(r) and is_total(r))
        is_total(r)
    }
}

theorem linear_order_relation_is_reflexive[T](r: (T, T) -> Bool) {
    is_linear_order_relation(r) implies is_reflexive(r)
} by {
    if is_linear_order_relation(r) {
        linear_order_relation_is_partial_order(r)
        partial_order_relation_is_reflexive(r)
        is_reflexive(r)
    }
}

theorem linear_order_relation_is_transitive[T](r: (T, T) -> Bool) {
    is_linear_order_relation(r) implies is_transitive(r)
} by {
    if is_linear_order_relation(r) {
        linear_order_relation_is_partial_order(r)
        partial_order_relation_is_transitive(r)
        is_transitive(r)
    }
}

theorem linear_order_relation_is_antisymmetric[T](r: (T, T) -> Bool) {
    is_linear_order_relation(r) implies is_antisymmetric(r)
} by {
    if is_linear_order_relation(r) {
        linear_order_relation_is_partial_order(r)
        partial_order_relation_is_antisymmetric(r)
        is_antisymmetric(r)
    }
}

theorem relation_strict_part_at[T](r: (T, T) -> Bool, a: T, b: T) {
    relation_strict_part(r, a, b) = (r(a, b) and a != b)
}

theorem relation_strict_part_eq_of_relation_eq[T](r: (T, T) -> Bool, s: (T, T) -> Bool) {
    r = s implies relation_strict_part(r) = relation_strict_part(s)
} by {
    if r = s {
        let u = relation_strict_part(r)
        let v = relation_strict_part(s)
        forall(a: T, b: T) {
            r(a, b) = s(a, b)
            u(a, b) = (r(a, b) and a != b)
            v(a, b) = (s(a, b) and a != b)
            u(a, b) = v(a, b)
        }
        binary_function_extensionality(u, v)
    }
}

theorem relation_converse_eq_of_relation_eq[A, B](r: (A, B) -> Bool, s: (A, B) -> Bool) {
    r = s implies relation_converse(r) = relation_converse(s)
} by {
    if r = s {
        let u = relation_converse(r)
        let v = relation_converse(s)
        forall(b: B, a: A) {
            r(a, b) = s(a, b)
            u(b, a) = r(a, b)
            v(b, a) = s(a, b)
            u(b, a) = v(b, a)
        }
        binary_function_extensionality(u, v)
    }
}

theorem relation_strict_part_is_irreflexive[T](r: (T, T) -> Bool) {
    is_irreflexive(relation_strict_part(r))
} by {
    forall(a: T) {
        if relation_strict_part(r, a, a) {
            a != a
            false
        }
    }
}

theorem relation_strict_part_is_asymmetric_of_partial_order_relation[T](r: (T, T) -> Bool) {
    is_partial_order_relation(r) implies is_asymmetric(relation_strict_part(r))
} by {
    if is_partial_order_relation(r) {
        partial_order_relation_is_antisymmetric(r)
        forall(a: T, b: T) {
            if relation_strict_part(r, a, b) {
                r(a, b)
                a != b
                if relation_strict_part(r, b, a) {
                    r(b, a)
                    a = b
                    false
                }
            }
        }
    }
}

theorem relation_strict_part_is_transitive_of_partial_order_relation[T](r: (T, T) -> Bool) {
    is_partial_order_relation(r) implies is_transitive(relation_strict_part(r))
} by {
    if is_partial_order_relation(r) {
        partial_order_relation_is_transitive(r)
        partial_order_relation_is_antisymmetric(r)
        forall(a: T, b: T, c: T) {
            if relation_strict_part(r, a, b) and relation_strict_part(r, b, c) {
                r(a, b)
                a != b
                r(b, c)
                b != c
                r(a, c)
                if a = c {
                    r(c, b)
                    r(b, c)
                    b = c
                    false
                }
                a != c
                relation_strict_part(r, a, c)
            }
        }
    }
}

theorem relation_converse_is_partial_order_relation[T](r: (T, T) -> Bool) {
    is_partial_order_relation(r) implies is_partial_order_relation(relation_converse(r))
} by {
    if is_partial_order_relation(r) {
        partial_order_relation_is_reflexive(r)
        relation_converse_is_reflexive(r)
        is_reflexive(relation_converse(r))
        partial_order_relation_is_transitive(r)
        relation_converse_is_transitive(r)
        is_transitive(relation_converse(r))
        partial_order_relation_is_antisymmetric(r)
        relation_converse_is_antisymmetric(r)
        is_antisymmetric(relation_converse(r))
        is_partial_order_relation(relation_converse(r))
    }
}

theorem relation_converse_is_linear_order_relation[T](r: (T, T) -> Bool) {
    is_linear_order_relation(r) implies is_linear_order_relation(relation_converse(r))
} by {
    if is_linear_order_relation(r) {
        linear_order_relation_is_partial_order(r)
        relation_converse_is_partial_order_relation(r)
        is_partial_order_relation(relation_converse(r))
        linear_order_relation_is_total(r)
        relation_converse_is_total(r)
        is_total(relation_converse(r))
        is_linear_order_relation(relation_converse(r))
    }
}

theorem partial_order_relation_eq_transport[T](r: (T, T) -> Bool, s: (T, T) -> Bool) {
    r = s and is_partial_order_relation(r) implies is_partial_order_relation(s)
} by {
    if r = s and is_partial_order_relation(r) {
        binary_function_eq_transport_predicate(is_partial_order_relation[T], r, s)
        is_partial_order_relation(s)
    }
}

theorem partial_order_relation_eq_transport_rev[T](r: (T, T) -> Bool, s: (T, T) -> Bool) {
    r = s and is_partial_order_relation(s) implies is_partial_order_relation(r)
} by {
    if r = s and is_partial_order_relation(s) {
        binary_function_eq_transport_predicate_rev(is_partial_order_relation[T], r, s)
        is_partial_order_relation(r)
    }
}

theorem partial_order_relation_eq_iff[T](r: (T, T) -> Bool, s: (T, T) -> Bool) {
    r = s implies is_partial_order_relation(r) = is_partial_order_relation(s)
} by {
    if r = s {
        if is_partial_order_relation(r) {
            partial_order_relation_eq_transport(r, s)
            is_partial_order_relation(s)
        }
        if is_partial_order_relation(s) {
            partial_order_relation_eq_transport_rev(r, s)
            is_partial_order_relation(r)
        }
        is_partial_order_relation(r) = is_partial_order_relation(s)
    }
}

theorem linear_order_relation_eq_transport[T](r: (T, T) -> Bool, s: (T, T) -> Bool) {
    r = s and is_linear_order_relation(r) implies is_linear_order_relation(s)
} by {
    if r = s and is_linear_order_relation(r) {
        binary_function_eq_transport_predicate(is_linear_order_relation[T], r, s)
        is_linear_order_relation(s)
    }
}

theorem linear_order_relation_eq_transport_rev[T](r: (T, T) -> Bool, s: (T, T) -> Bool) {
    r = s and is_linear_order_relation(s) implies is_linear_order_relation(r)
} by {
    if r = s and is_linear_order_relation(s) {
        binary_function_eq_transport_predicate_rev(is_linear_order_relation[T], r, s)
        is_linear_order_relation(r)
    }
}

theorem linear_order_relation_eq_iff[T](r: (T, T) -> Bool, s: (T, T) -> Bool) {
    r = s implies is_linear_order_relation(r) = is_linear_order_relation(s)
} by {
    if r = s {
        if is_linear_order_relation(r) {
            linear_order_relation_eq_transport(r, s)
            is_linear_order_relation(s)
        }
        if is_linear_order_relation(s) {
            linear_order_relation_eq_transport_rev(r, s)
            is_linear_order_relation(r)
        }
        is_linear_order_relation(r) = is_linear_order_relation(s)
    }
}

theorem lte_relation_is_reflexive[P: PartialOrder] {
    is_reflexive(P.lte)
} by {
    forall(a: P) {
        lte_refl(a)
        a <= a
    }
}

theorem lte_relation_is_transitive[P: PartialOrder] {
    is_transitive(P.lte)
} by {
    forall(a: P, b: P, c: P) {
        if a <= b and b <= c {
            lte_trans(a, b, c)
            a <= c
        }
    }
}

theorem lt_relation_is_transitive[P: PartialOrder] {
    is_transitive(P.lt)
} by {
    forall(a: P, b: P, c: P) {
        if a < b and b < c {
            lt_trans(a, b, c)
            a < c
        }
    }
}

theorem gte_relation_is_reflexive[P: PartialOrder] {
    is_reflexive(P.gte)
} by {
    forall(a: P) {
        gte_refl(a)
        a >= a
    }
}

theorem gte_relation_is_transitive[P: PartialOrder] {
    is_transitive(P.gte)
} by {
    forall(a: P, b: P, c: P) {
        if a >= b and b >= c {
            gte_trans(a, b, c)
            a >= c
        }
    }
}

theorem lte_relation_is_antisymmetric[P: PartialOrder] {
    is_antisymmetric(P.lte)
} by {
    forall(a: P, b: P) {
        if a <= b and b <= a {
            lte_antisymm(a, b)
            a = b
        }
    }
}

theorem lte_relation_is_partial_order_relation[P: PartialOrder] {
    is_partial_order_relation(P.lte)
} by {
    lte_relation_is_reflexive[P]
    lte_relation_is_transitive[P]
    lte_relation_is_antisymmetric[P]
    is_partial_order_relation(P.lte)
}

theorem gte_relation_is_antisymmetric[P: PartialOrder] {
    is_antisymmetric(P.gte)
} by {
    forall(a: P, b: P) {
        if a >= b and b >= a {
            a >= b
            b <= a
            b >= a
            a <= b
            lte_antisymm(a, b)
            a = b
        }
    }
}

theorem gte_relation_is_partial_order_relation[P: PartialOrder] {
    is_partial_order_relation(P.gte)
} by {
    gte_relation_is_reflexive[P]
    gte_relation_is_transitive[P]
    gte_relation_is_antisymmetric[P]
    is_partial_order_relation(P.gte)
}

theorem lt_relation_is_irreflexive[P: PartialOrder] {
    is_irreflexive(P.lt)
} by {
    forall(a: P) {
        not_lt_ref(a)
    }
}

theorem lt_relation_strict_part_lte[P: PartialOrder] {
    P.lt = relation_strict_part(P.lte)
} by {
    let u = P.lt
    let v = relation_strict_part(P.lte)
    forall(a: P, b: P) {
        u(a, b) = P.lt(a, b)
        P.lt(a, b) = (a <= b and a != b)
        v(a, b) = (P.lte(a, b) and a != b)
        u(a, b) = v(a, b)
    }
    binary_function_extensionality(u, v)
}

theorem gt_relation_strict_part_gte[P: PartialOrder] {
    P.gt = relation_strict_part(P.gte)
} by {
    let u = P.gt
    let v = relation_strict_part(P.gte)
    forall(a: P, b: P) {
        u(a, b) = P.gt(a, b)
        P.gt(a, b) = (a >= b and a != b)
        v(a, b) = (P.gte(a, b) and a != b)
        u(a, b) = v(a, b)
    }
    binary_function_extensionality(u, v)
}

theorem gt_relation_is_transitive[P: PartialOrder] {
    is_transitive(P.gt)
} by {
    forall(a: P, b: P, c: P) {
        if a > b and b > c {
            gt_trans(a, b, c)
            a > c
        }
    }
}

theorem gt_relation_is_irreflexive[P: PartialOrder] {
    is_irreflexive(P.gt)
} by {
    forall(a: P) {
        not_gt_ref(a)
    }
}

theorem lt_relation_is_asymmetric[P: PartialOrder] {
    is_asymmetric(P.lt)
} by {
    forall(a: P, b: P) {
        if a < b {
            if b < a {
                lt_trans(a, b, a)
                a < a
                not_lt_ref(a)
                false
            }
        }
    }
}

theorem gt_relation_is_asymmetric[P: PartialOrder] {
    is_asymmetric(P.gt)
} by {
    forall(a: P, b: P) {
        if a > b {
            if b > a {
                gt_trans(a, b, a)
                a > a
                not_gt_ref(a)
                false
            }
        }
    }
}

theorem lt_relation_subset_lte[P: PartialOrder] {
    relation_subset(P.lt, P.lte)
} by {
    forall(a: P, b: P) {
        if a < b {
            a <= b
        }
    }
}

theorem gt_relation_subset_gte[P: PartialOrder] {
    relation_subset(P.gt, P.gte)
} by {
    forall(a: P, b: P) {
        if a > b {
            a >= b
        }
    }
}

theorem lte_relation_is_total[L: LinearOrder] {
    is_total(L.lte)
} by {
    forall(a: L, b: L) {
        lte_or_gte(a, b)
        a <= b or a >= b
    }
}

theorem lte_relation_is_linear_order_relation[L: LinearOrder] {
    is_linear_order_relation(L.lte)
} by {
    lte_relation_is_partial_order_relation[L]
    lte_relation_is_total[L]
    is_linear_order_relation(L.lte)
}

theorem gte_relation_is_total[L: LinearOrder] {
    is_total(L.gte)
} by {
    forall(a: L, b: L) {
        lte_or_gte(a, b)
        a <= b or a >= b
        if a <= b {
            b >= a
            a >= b or b >= a
        } else {
            a >= b
            a >= b or b >= a
        }
    }
}

theorem gte_relation_is_linear_order_relation[L: LinearOrder] {
    is_linear_order_relation(L.gte)
} by {
    gte_relation_is_partial_order_relation[L]
    gte_relation_is_total[L]
    is_linear_order_relation(L.gte)
}

theorem lte_relation_converse[P: PartialOrder] {
    relation_converse(P.lte) = P.gte
} by {
    let u = relation_converse(P.lte)
    let v = P.gte
    forall(a: P, b: P) {
        u(a, b) = P.lte(b, a)
        v(a, b) = P.gte(a, b)
        P.gte(a, b) = P.lte(b, a)
        u(a, b) = v(a, b)
    }
    binary_function_extensionality(u, v)
}

theorem gte_relation_converse[P: PartialOrder] {
    relation_converse(P.gte) = P.lte
} by {
    let u = relation_converse(P.gte)
    let v = P.lte
    forall(a: P, b: P) {
        u(a, b) = P.gte(b, a)
        P.gte(b, a) = P.lte(a, b)
        v(a, b) = P.lte(a, b)
        u(a, b) = v(a, b)
    }
    binary_function_extensionality(u, v)
}

theorem lt_relation_converse[P: PartialOrder] {
    relation_converse(P.lt) = P.gt
} by {
    let u = relation_converse(P.lt)
    let v = P.gt
    forall(a: P, b: P) {
        u(a, b) = P.lt(b, a)
        v(a, b) = P.gt(a, b)
        P.gt(a, b) = P.lt(b, a)
        u(a, b) = v(a, b)
    }
    binary_function_extensionality(u, v)
}

theorem gt_relation_converse[P: PartialOrder] {
    relation_converse(P.gt) = P.lt
} by {
    let u = relation_converse(P.gt)
    let v = P.lt
    forall(a: P, b: P) {
        u(a, b) = P.gt(b, a)
        P.gt(b, a) = P.lt(a, b)
        v(a, b) = P.lt(a, b)
        u(a, b) = v(a, b)
    }
    binary_function_extensionality(u, v)
}

theorem lte_refl_trans_closure_at[P: PartialOrder](a: P, b: P) {
    relation_refl_trans_closure(P.lte, a, b) = (a <= b)
} by {
    if relation_refl_trans_closure(P.lte, a, b) {
        lte_relation_is_reflexive[P]
        lte_relation_is_transitive[P]
        relation_subset_refl(P.lte)
        relation_refl_trans_closure_subset_of_reflexive_transitive(P.lte, P.lte)
        relation_subset(relation_refl_trans_closure(P.lte), P.lte)
        relation_subset_step(relation_refl_trans_closure(P.lte), P.lte, a, b)
        a <= b
    }
    if a <= b {
        relation_subset_refl_trans_closure(P.lte)
        relation_subset_step(P.lte, relation_refl_trans_closure(P.lte), a, b)
        relation_refl_trans_closure(P.lte, a, b)
    }
    relation_refl_trans_closure(P.lte, a, b) = (a <= b)
}

theorem lte_refl_trans_closure[P: PartialOrder] {
    relation_refl_trans_closure(P.lte) = P.lte
} by {
    let u = relation_refl_trans_closure(P.lte)
    let v = P.lte
    forall(a: P, b: P) {
        lte_refl_trans_closure_at(a, b)
        u(a, b) = v(a, b)
    }
    binary_function_extensionality(u, v)
}

theorem gte_refl_trans_closure_at[P: PartialOrder](a: P, b: P) {
    relation_refl_trans_closure(P.gte, a, b) = (a >= b)
} by {
    if relation_refl_trans_closure(P.gte, a, b) {
        gte_relation_is_reflexive[P]
        gte_relation_is_transitive[P]
        relation_subset_refl(P.gte)
        relation_refl_trans_closure_subset_of_reflexive_transitive(P.gte, P.gte)
        relation_subset(relation_refl_trans_closure(P.gte), P.gte)
        relation_subset_step(relation_refl_trans_closure(P.gte), P.gte, a, b)
        a >= b
    }
    if a >= b {
        relation_subset_refl_trans_closure(P.gte)
        relation_subset_step(P.gte, relation_refl_trans_closure(P.gte), a, b)
        relation_refl_trans_closure(P.gte, a, b)
    }
    relation_refl_trans_closure(P.gte, a, b) = (a >= b)
}

theorem gte_refl_trans_closure[P: PartialOrder] {
    relation_refl_trans_closure(P.gte) = P.gte
} by {
    let u = relation_refl_trans_closure(P.gte)
    let v = P.gte
    forall(a: P, b: P) {
        gte_refl_trans_closure_at(a, b)
        u(a, b) = v(a, b)
    }
    binary_function_extensionality(u, v)
}

theorem lte_trans_closure_at[P: PartialOrder](a: P, b: P) {
    relation_trans_closure(P.lte, a, b) = (a <= b)
} by {
    if relation_trans_closure(P.lte, a, b) {
        lte_relation_is_transitive[P]
        relation_subset_refl(P.lte)
        relation_trans_closure_subset_of_transitive(P.lte, P.lte)
        relation_subset(relation_trans_closure(P.lte), P.lte)
        relation_subset_step(relation_trans_closure(P.lte), P.lte, a, b)
        a <= b
    }
    if a <= b {
        relation_subset_trans_closure(P.lte)
        relation_subset_step(P.lte, relation_trans_closure(P.lte), a, b)
        relation_trans_closure(P.lte, a, b)
    }
    relation_trans_closure(P.lte, a, b) = (a <= b)
}

theorem lte_trans_closure[P: PartialOrder] {
    relation_trans_closure(P.lte) = P.lte
} by {
    let u = relation_trans_closure(P.lte)
    let v = P.lte
    forall(a: P, b: P) {
        lte_trans_closure_at(a, b)
        u(a, b) = v(a, b)
    }
    binary_function_extensionality(u, v)
}

theorem gte_trans_closure_at[P: PartialOrder](a: P, b: P) {
    relation_trans_closure(P.gte, a, b) = (a >= b)
} by {
    if relation_trans_closure(P.gte, a, b) {
        gte_relation_is_transitive[P]
        relation_subset_refl(P.gte)
        relation_trans_closure_subset_of_transitive(P.gte, P.gte)
        relation_subset(relation_trans_closure(P.gte), P.gte)
        relation_subset_step(relation_trans_closure(P.gte), P.gte, a, b)
        a >= b
    }
    if a >= b {
        relation_subset_trans_closure(P.gte)
        relation_subset_step(P.gte, relation_trans_closure(P.gte), a, b)
        relation_trans_closure(P.gte, a, b)
    }
    relation_trans_closure(P.gte, a, b) = (a >= b)
}

theorem gte_trans_closure[P: PartialOrder] {
    relation_trans_closure(P.gte) = P.gte
} by {
    let u = relation_trans_closure(P.gte)
    let v = P.gte
    forall(a: P, b: P) {
        gte_trans_closure_at(a, b)
        u(a, b) = v(a, b)
    }
    binary_function_extensionality(u, v)
}

theorem lt_trans_closure_at[P: PartialOrder](a: P, b: P) {
    relation_trans_closure(P.lt, a, b) = (a < b)
} by {
    if relation_trans_closure(P.lt, a, b) {
        lt_relation_is_transitive[P]
        relation_subset_refl(P.lt)
        relation_trans_closure_subset_of_transitive(P.lt, P.lt)
        relation_subset(relation_trans_closure(P.lt), P.lt)
        relation_subset_step(relation_trans_closure(P.lt), P.lt, a, b)
        a < b
    }
    if a < b {
        relation_subset_trans_closure(P.lt)
        relation_subset_step(P.lt, relation_trans_closure(P.lt), a, b)
        relation_trans_closure(P.lt, a, b)
    }
    relation_trans_closure(P.lt, a, b) = (a < b)
}

theorem lt_trans_closure[P: PartialOrder] {
    relation_trans_closure(P.lt) = P.lt
} by {
    let u = relation_trans_closure(P.lt)
    let v = P.lt
    forall(a: P, b: P) {
        lt_trans_closure_at(a, b)
        u(a, b) = v(a, b)
    }
    binary_function_extensionality(u, v)
}

theorem gt_trans_closure_at[P: PartialOrder](a: P, b: P) {
    relation_trans_closure(P.gt, a, b) = (a > b)
} by {
    if relation_trans_closure(P.gt, a, b) {
        gt_relation_is_transitive[P]
        relation_subset_refl(P.gt)
        relation_trans_closure_subset_of_transitive(P.gt, P.gt)
        relation_subset(relation_trans_closure(P.gt), P.gt)
        relation_subset_step(relation_trans_closure(P.gt), P.gt, a, b)
        a > b
    }
    if a > b {
        relation_subset_trans_closure(P.gt)
        relation_subset_step(P.gt, relation_trans_closure(P.gt), a, b)
        relation_trans_closure(P.gt, a, b)
    }
    relation_trans_closure(P.gt, a, b) = (a > b)
}

theorem gt_trans_closure[P: PartialOrder] {
    relation_trans_closure(P.gt) = P.gt
} by {
    let u = relation_trans_closure(P.gt)
    let v = P.gt
    forall(a: P, b: P) {
        gt_trans_closure_at(a, b)
        u(a, b) = v(a, b)
    }
    binary_function_extensionality(u, v)
}

theorem lte_pullback_is_reflexive[A, B: PartialOrder](f: A -> B) {
    is_reflexive(relation_pullback(f, B.lte))
} by {
    lte_relation_is_reflexive[B]
    relation_pullback_is_reflexive(f, B.lte)
    is_reflexive(relation_pullback(f, B.lte))
}

theorem gte_pullback_is_reflexive[A, B: PartialOrder](f: A -> B) {
    is_reflexive(relation_pullback(f, B.gte))
} by {
    gte_relation_is_reflexive[B]
    relation_pullback_is_reflexive(f, B.gte)
    is_reflexive(relation_pullback(f, B.gte))
}

theorem lte_pullback_is_transitive[A, B: PartialOrder](f: A -> B) {
    is_transitive(relation_pullback(f, B.lte))
} by {
    lte_relation_is_transitive[B]
    relation_pullback_is_transitive(f, B.lte)
    is_transitive(relation_pullback(f, B.lte))
}

theorem gte_pullback_is_transitive[A, B: PartialOrder](f: A -> B) {
    is_transitive(relation_pullback(f, B.gte))
} by {
    gte_relation_is_transitive[B]
    relation_pullback_is_transitive(f, B.gte)
    is_transitive(relation_pullback(f, B.gte))
}

theorem lte_pullback_is_antisymmetric_of_injective[A, B: PartialOrder](f: A -> B) {
    is_injective_fn(f) implies is_antisymmetric(relation_pullback(f, B.lte))
} by {
    if is_injective_fn(f) {
        lte_relation_is_antisymmetric[B]
        relation_pullback_is_antisymmetric_of_injective(f, B.lte)
        is_antisymmetric(relation_pullback(f, B.lte))
    }
}

theorem gte_pullback_is_antisymmetric_of_injective[A, B: PartialOrder](f: A -> B) {
    is_injective_fn(f) implies is_antisymmetric(relation_pullback(f, B.gte))
} by {
    if is_injective_fn(f) {
        gte_relation_is_antisymmetric[B]
        relation_pullback_is_antisymmetric_of_injective(f, B.gte)
        is_antisymmetric(relation_pullback(f, B.gte))
    }
}

theorem lte_pullback_is_order_data_of_bijection_map[A, B: PartialOrder](e: Bijection[A, B]) {
    is_reflexive(relation_pullback(e.map, B.lte)) and
    is_transitive(relation_pullback(e.map, B.lte)) and
    is_antisymmetric(relation_pullback(e.map, B.lte))
} by {
    lte_pullback_is_reflexive(e.map)
    lte_pullback_is_transitive(e.map)
    bijection_map_is_injective(e)
    lte_pullback_is_antisymmetric_of_injective(e.map)
    is_reflexive(relation_pullback(e.map, B.lte)) and
    is_transitive(relation_pullback(e.map, B.lte)) and
    is_antisymmetric(relation_pullback(e.map, B.lte))
}

theorem gte_pullback_is_order_data_of_bijection_map[A, B: PartialOrder](e: Bijection[A, B]) {
    is_reflexive(relation_pullback(e.map, B.gte)) and
    is_transitive(relation_pullback(e.map, B.gte)) and
    is_antisymmetric(relation_pullback(e.map, B.gte))
} by {
    gte_pullback_is_reflexive(e.map)
    gte_pullback_is_transitive(e.map)
    bijection_map_is_injective(e)
    gte_pullback_is_antisymmetric_of_injective(e.map)
    is_reflexive(relation_pullback(e.map, B.gte)) and
    is_transitive(relation_pullback(e.map, B.gte)) and
    is_antisymmetric(relation_pullback(e.map, B.gte))
}

theorem lt_pullback_is_irreflexive[A, B: PartialOrder](f: A -> B) {
    is_irreflexive(relation_pullback(f, B.lt))
} by {
    lt_relation_is_irreflexive[B]
    relation_pullback_is_irreflexive(f, B.lt)
    is_irreflexive(relation_pullback(f, B.lt))
}

theorem gt_pullback_is_irreflexive[A, B: PartialOrder](f: A -> B) {
    is_irreflexive(relation_pullback(f, B.gt))
} by {
    gt_relation_is_irreflexive[B]
    relation_pullback_is_irreflexive(f, B.gt)
    is_irreflexive(relation_pullback(f, B.gt))
}

theorem lt_pullback_is_asymmetric[A, B: PartialOrder](f: A -> B) {
    is_asymmetric(relation_pullback(f, B.lt))
} by {
    lt_relation_is_asymmetric[B]
    relation_pullback_is_asymmetric(f, B.lt)
    is_asymmetric(relation_pullback(f, B.lt))
}

theorem gt_pullback_is_asymmetric[A, B: PartialOrder](f: A -> B) {
    is_asymmetric(relation_pullback(f, B.gt))
} by {
    gt_relation_is_asymmetric[B]
    relation_pullback_is_asymmetric(f, B.gt)
    is_asymmetric(relation_pullback(f, B.gt))
}

theorem lt_pullback_is_strict_order_data_of_bijection_map[A, B: PartialOrder](e: Bijection[A, B]) {
    is_irreflexive(relation_pullback(e.map, B.lt)) and
    is_asymmetric(relation_pullback(e.map, B.lt)) and
    is_transitive(relation_pullback(e.map, B.lt))
} by {
    lt_pullback_is_irreflexive(e.map)
    lt_pullback_is_asymmetric(e.map)
    lt_relation_is_transitive[B]
    relation_pullback_is_transitive(e.map, B.lt)
    is_transitive(relation_pullback(e.map, B.lt))
    is_irreflexive(relation_pullback(e.map, B.lt)) and
    is_asymmetric(relation_pullback(e.map, B.lt)) and
    is_transitive(relation_pullback(e.map, B.lt))
}

theorem gt_pullback_is_strict_order_data_of_bijection_map[A, B: PartialOrder](e: Bijection[A, B]) {
    is_irreflexive(relation_pullback(e.map, B.gt)) and
    is_asymmetric(relation_pullback(e.map, B.gt)) and
    is_transitive(relation_pullback(e.map, B.gt))
} by {
    gt_pullback_is_irreflexive(e.map)
    gt_pullback_is_asymmetric(e.map)
    gt_relation_is_transitive[B]
    relation_pullback_is_transitive(e.map, B.gt)
    is_transitive(relation_pullback(e.map, B.gt))
    is_irreflexive(relation_pullback(e.map, B.gt)) and
    is_asymmetric(relation_pullback(e.map, B.gt)) and
    is_transitive(relation_pullback(e.map, B.gt))
}

theorem lte_pullback_is_total[A, B: LinearOrder](f: A -> B) {
    is_total(relation_pullback(f, B.lte))
} by {
    lte_relation_is_total[B]
    relation_pullback_is_total(f, B.lte)
    is_total(relation_pullback(f, B.lte))
}

theorem gte_pullback_is_total[A, B: LinearOrder](f: A -> B) {
    is_total(relation_pullback(f, B.gte))
} by {
    gte_relation_is_total[B]
    relation_pullback_is_total(f, B.gte)
    is_total(relation_pullback(f, B.gte))
}

theorem lte_pullback_is_total_of_bijection_map[A, B: LinearOrder](e: Bijection[A, B]) {
    is_total(relation_pullback(e.map, B.lte))
} by {
    lte_pullback_is_total(e.map)
}

theorem gte_pullback_is_total_of_bijection_map[A, B: LinearOrder](e: Bijection[A, B]) {
    is_total(relation_pullback(e.map, B.gte))
} by {
    gte_pullback_is_total(e.map)
}

theorem lt_refl_trans_closure_at[L: LinearOrder](a: L, b: L) {
    relation_refl_trans_closure(L.lt, a, b) = (a <= b)
} by {
    if relation_refl_trans_closure(L.lt, a, b) {
        relation_refl_trans_closure_eq_eq_or_trans_closure(L.lt, a, b)
        if a = b {
            a <= b
        } else {
            lt_trans_closure_at(a, b)
            a < b
            a <= b
        }
    }
    if a <= b {
        not_gt_ref(b)
        not (a > b)
        lt_or_eq_of_not_gt(a, b)
        if a < b {
            relation_subset_refl_trans_closure(L.lt)
            relation_subset_step(L.lt, relation_refl_trans_closure(L.lt), a, b)
            relation_refl_trans_closure(L.lt, a, b)
        } else {
            relation_refl_trans_closure_is_reflexive(L.lt)
            reflexive_self(relation_refl_trans_closure(L.lt), a)
            relation_refl_trans_closure(L.lt, a, a)
            relation_refl_trans_closure(L.lt, a, b)
        }
    }
    relation_refl_trans_closure(L.lt, a, b) = (a <= b)
}

theorem lt_refl_trans_closure[L: LinearOrder] {
    relation_refl_trans_closure(L.lt) = L.lte
} by {
    let u = relation_refl_trans_closure(L.lt)
    let v = L.lte
    forall(a: L, b: L) {
        lt_refl_trans_closure_at(a, b)
        u(a, b) = v(a, b)
    }
    binary_function_extensionality(u, v)
}

theorem gt_refl_trans_closure_at[L: LinearOrder](a: L, b: L) {
    relation_refl_trans_closure(L.gt, a, b) = (a >= b)
} by {
    if relation_refl_trans_closure(L.gt, a, b) {
        relation_refl_trans_closure_eq_eq_or_trans_closure(L.gt, a, b)
        if a = b {
            a >= b
        } else {
            gt_trans_closure_at(a, b)
            a > b
            a >= b
        }
    }
    if a >= b {
        not_lt_ref(b)
        not (a < b)
        gt_or_eq_of_not_lt(a, b)
        if a > b {
            relation_subset_refl_trans_closure(L.gt)
            relation_subset_step(L.gt, relation_refl_trans_closure(L.gt), a, b)
            relation_refl_trans_closure(L.gt, a, b)
        } else {
            relation_refl_trans_closure_is_reflexive(L.gt)
            reflexive_self(relation_refl_trans_closure(L.gt), a)
            relation_refl_trans_closure(L.gt, a, a)
            relation_refl_trans_closure(L.gt, a, b)
        }
    }
    relation_refl_trans_closure(L.gt, a, b) = (a >= b)
}

theorem gt_refl_trans_closure[L: LinearOrder] {
    relation_refl_trans_closure(L.gt) = L.gte
} by {
    let u = relation_refl_trans_closure(L.gt)
    let v = L.gte
    forall(a: L, b: L) {
        gt_refl_trans_closure_at(a, b)
        u(a, b) = v(a, b)
    }
    binary_function_extensionality(u, v)
}

theorem lte_pullback_refl_trans_closure_of_bijection[A, B: PartialOrder](f: A -> B) {
    is_bijection_fn(f) implies relation_refl_trans_closure(relation_pullback(f, B.lte)) = relation_pullback(f, B.lte)
} by {
    if is_bijection_fn(f) {
        relation_refl_trans_closure_pullback_of_bijection(f, B.lte)
        relation_pullback(f, relation_refl_trans_closure(B.lte)) = relation_refl_trans_closure(relation_pullback(f, B.lte))
        lte_refl_trans_closure[B]
        relation_pullback(f, relation_refl_trans_closure(B.lte)) = relation_pullback(f, B.lte)
        relation_refl_trans_closure(relation_pullback(f, B.lte)) = relation_pullback(f, B.lte)
    }
}

theorem gte_pullback_refl_trans_closure_of_bijection[A, B: PartialOrder](f: A -> B) {
    is_bijection_fn(f) implies relation_refl_trans_closure(relation_pullback(f, B.gte)) = relation_pullback(f, B.gte)
} by {
    if is_bijection_fn(f) {
        relation_refl_trans_closure_pullback_of_bijection(f, B.gte)
        relation_pullback(f, relation_refl_trans_closure(B.gte)) = relation_refl_trans_closure(relation_pullback(f, B.gte))
        gte_refl_trans_closure[B]
        relation_pullback(f, relation_refl_trans_closure(B.gte)) = relation_pullback(f, B.gte)
        relation_refl_trans_closure(relation_pullback(f, B.gte)) = relation_pullback(f, B.gte)
    }
}

theorem lte_pullback_trans_closure_of_surjective[A, B: PartialOrder](f: A -> B) {
    is_surjective_fn(f) implies relation_trans_closure(relation_pullback(f, B.lte)) = relation_pullback(f, B.lte)
} by {
    if is_surjective_fn(f) {
        relation_trans_closure_pullback_of_surjective(f, B.lte)
        relation_pullback(f, relation_trans_closure(B.lte)) = relation_trans_closure(relation_pullback(f, B.lte))
        lte_trans_closure[B]
        relation_pullback(f, relation_trans_closure(B.lte)) = relation_pullback(f, B.lte)
        relation_trans_closure(relation_pullback(f, B.lte)) = relation_pullback(f, B.lte)
    }
}

theorem gte_pullback_trans_closure_of_surjective[A, B: PartialOrder](f: A -> B) {
    is_surjective_fn(f) implies relation_trans_closure(relation_pullback(f, B.gte)) = relation_pullback(f, B.gte)
} by {
    if is_surjective_fn(f) {
        relation_trans_closure_pullback_of_surjective(f, B.gte)
        relation_pullback(f, relation_trans_closure(B.gte)) = relation_trans_closure(relation_pullback(f, B.gte))
        gte_trans_closure[B]
        relation_pullback(f, relation_trans_closure(B.gte)) = relation_pullback(f, B.gte)
        relation_trans_closure(relation_pullback(f, B.gte)) = relation_pullback(f, B.gte)
    }
}

theorem lt_pullback_trans_closure_of_surjective[A, B: PartialOrder](f: A -> B) {
    is_surjective_fn(f) implies relation_trans_closure(relation_pullback(f, B.lt)) = relation_pullback(f, B.lt)
} by {
    if is_surjective_fn(f) {
        relation_trans_closure_pullback_of_surjective(f, B.lt)
        relation_pullback(f, relation_trans_closure(B.lt)) = relation_trans_closure(relation_pullback(f, B.lt))
        lt_trans_closure[B]
        relation_pullback(f, relation_trans_closure(B.lt)) = relation_pullback(f, B.lt)
        relation_trans_closure(relation_pullback(f, B.lt)) = relation_pullback(f, B.lt)
    }
}

theorem gt_pullback_trans_closure_of_surjective[A, B: PartialOrder](f: A -> B) {
    is_surjective_fn(f) implies relation_trans_closure(relation_pullback(f, B.gt)) = relation_pullback(f, B.gt)
} by {
    if is_surjective_fn(f) {
        relation_trans_closure_pullback_of_surjective(f, B.gt)
        relation_pullback(f, relation_trans_closure(B.gt)) = relation_trans_closure(relation_pullback(f, B.gt))
        gt_trans_closure[B]
        relation_pullback(f, relation_trans_closure(B.gt)) = relation_pullback(f, B.gt)
        relation_trans_closure(relation_pullback(f, B.gt)) = relation_pullback(f, B.gt)
    }
}

theorem lt_pullback_refl_trans_closure_of_bijection[A, B: LinearOrder](f: A -> B) {
    is_bijection_fn(f) implies relation_refl_trans_closure(relation_pullback(f, B.lt)) = relation_pullback(f, B.lte)
} by {
    if is_bijection_fn(f) {
        relation_refl_trans_closure_pullback_of_bijection(f, B.lt)
        relation_pullback(f, relation_refl_trans_closure(B.lt)) = relation_refl_trans_closure(relation_pullback(f, B.lt))
        lt_refl_trans_closure[B]
        relation_pullback(f, relation_refl_trans_closure(B.lt)) = relation_pullback(f, B.lte)
        relation_refl_trans_closure(relation_pullback(f, B.lt)) = relation_pullback(f, B.lte)
    }
}

theorem gt_pullback_refl_trans_closure_of_bijection[A, B: LinearOrder](f: A -> B) {
    is_bijection_fn(f) implies relation_refl_trans_closure(relation_pullback(f, B.gt)) = relation_pullback(f, B.gte)
} by {
    if is_bijection_fn(f) {
        relation_refl_trans_closure_pullback_of_bijection(f, B.gt)
        relation_pullback(f, relation_refl_trans_closure(B.gt)) = relation_refl_trans_closure(relation_pullback(f, B.gt))
        gt_refl_trans_closure[B]
        relation_pullback(f, relation_refl_trans_closure(B.gt)) = relation_pullback(f, B.gte)
        relation_refl_trans_closure(relation_pullback(f, B.gt)) = relation_pullback(f, B.gte)
    }
}
