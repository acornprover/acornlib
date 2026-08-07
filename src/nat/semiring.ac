from algebra.monoid.monoid import Monoid
from nat.nat_monoid import Nat
from semiring import Semiring

// This file contains theorems around how the natural numbers are contained in semirings.

theorem semiring_zero_pow[S: Semiring](n: Nat) {
    n != Nat.0 implies S.0.pow(n) = S.0
}

/// Standard embedding of the natural numbers into a semiring.
define from_nat[S: Semiring](n: Nat) -> S {
    match n {
        Nat.zero {
            S.0
        }
        Nat.suc(pred) {
            from_nat[S](pred) + S.1
        }
    }
}

/// from_nat maps 0 to the zero of the semiring.
theorem from_nat_zero[S: Semiring] {
    from_nat[S](Nat.0) = S.0
}

/// from_nat maps 1 to the one of the semiring.
theorem from_nat_one[S: Semiring] {
    from_nat[S](Nat.1) = S.1
}

/// from_nat preserves addition.
theorem from_nat_add[S: Semiring](m: Nat, n: Nat) {
    from_nat[S](m + n) = from_nat[S](m) + from_nat[S](n)
} by {
    define f(x: Nat) -> Bool {
        from_nat[S](m + x) = from_nat[S](m) + from_nat[S](x)
    }
    f(Nat.0)
    forall(x: Nat) {
        if f(x) {
            from_nat[S](m + x.suc) = from_nat[S](m + x) + S.1
            from_nat[S](m + x.suc) = from_nat[S](m) + from_nat[S](x) + S.1
            f(x.suc)
        }
    }
}

/// from_nat preserves multiplication.
theorem from_nat_mul[S: Semiring](m: Nat, n: Nat) {
    from_nat[S](m * n) = from_nat[S](m) * from_nat[S](n)
} by {
    define f(x: Nat) -> Bool {
        from_nat[S](m * x) = from_nat[S](m) * from_nat[S](x)
    }
    f(Nat.0)
    forall(x: Nat) {
        if f(x) {
            from_nat[S](m * x.suc) = from_nat[S](m * x + m)
            from_nat[S](m * x.suc) = from_nat[S](m * x) + from_nat[S](m)
            from_nat[S](m * x.suc) = from_nat[S](m) * from_nat[S](x) + from_nat[S](m)
            from_nat[S](m * x.suc) = from_nat[S](m) * (from_nat[S](x) + S.1)
            f(x.suc)
        }
    }
}
