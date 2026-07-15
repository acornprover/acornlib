from nat.nat_base import Nat, lte_mul_both, lte_mul, lt_or_lte, mul_cancel_left, lte_trans, mul_one_right, lte_antisymm, lt_not_ref, zero_or_suc, suc_sub_one, strong_induction, true_below, true_below_apply, only_zero_lte_zero, lte_cancel_suc, lt_imp_lte_suc, mod_mul, mul_to_zero
from nat.division import div_mod_decomp, div_mul
numerals Nat

/// For a base `p > 1`, dividing a nonzero number strictly decreases it.
theorem div_lt(n: Nat, p: Nat) {
    n != Nat.0 and Nat.1 < p implies n.div(p) < n
} by {
    if n != Nat.0 and Nat.1 < p {
        div_mod_decomp(n, p)
        if n <= n.div(p) {
            lte_mul_both(p, n, n.div(p))
            n * p <= n.div(p) * p
            lte_trans(n * p, n.div(p) * p, n)
            n * p <= n
            lte_mul(n, p)
            lte_antisymm(n * p, n)
            n * p = n
            mul_one_right(n)
            mul_cancel_left(n, p, Nat.1)
            p = Nat.1
            lt_not_ref(Nat.1)
            false
        }
        lt_or_lte(n.div(p), n)
        n.div(p) < n
    }
}

/// The sum of the base-`p` digits of `n`, computed with an explicit recursion
/// budget `fuel`. Any `fuel >= n` gives the true digit sum.
define digit_sum_fuel(p: Nat, n: Nat, fuel: Nat) -> Nat {
    match fuel {
        Nat.zero {
            Nat.0
        }
        Nat.suc(k) {
            if n = Nat.0 {
                Nat.0
            } else {
                n.mod(p) + digit_sum_fuel(p, n.div(p), k)
            }
        }
    }
}

/// The sum of the base-`p` digits of `n`.
define digit_sum(p: Nat, n: Nat) -> Nat {
    digit_sum_fuel(p, n, n)
}

/// True if a recursion budget is large enough to give the true digit sum.
define digit_sum_fuel_large_case(p: Nat, n: Nat, fuel: Nat) -> Bool {
    Nat.1 < p and n <= fuel implies digit_sum_fuel(p, n, fuel) = digit_sum(p, n)
}

/// The digit sum of zero vanishes, for any recursion budget.
theorem digit_sum_fuel_at_zero(p: Nat, fuel: Nat) {
    digit_sum_fuel(p, Nat.0, fuel) = Nat.0
} by {
    zero_or_suc(fuel)
}

/// One step of the digit-sum recursion peels off the lowest digit.
theorem digit_sum_fuel_step(p: Nat, n: Nat, k: Nat) {
    n != Nat.0 implies
        digit_sum_fuel(p, n, k.suc) = n.mod(p) + digit_sum_fuel(p, n.div(p), k)
}

/// The digit sum of a nonzero number splits into its final digit and quotient.
theorem digit_sum_step(p: Nat, n: Nat) {
    n != Nat.0 implies
        digit_sum(p, n) = n.mod(p) + digit_sum_fuel(p, n.div(p), n - Nat.1)
} by {
    if n != Nat.0 {
        zero_or_suc(n)
        if n = Nat.0 {
            false
        }
        let k: Nat satisfy { n = k.suc }
        suc_sub_one(k)
        digit_sum_fuel_step(p, n, k)
        digit_sum(p, n) = n.mod(p) + digit_sum_fuel(p, n.div(p), n - Nat.1)
    }
}

/// Oversized recursion budgets give the same digit sum.
theorem digit_sum_fuel_large(p: Nat, n: Nat, fuel: Nat) {
    Nat.1 < p and n <= fuel implies digit_sum_fuel(p, n, fuel) = digit_sum(p, n)
} by {
    let f: Nat -> Bool = function(m: Nat) {
        forall(fuel2: Nat) {
            digit_sum_fuel_large_case(p, m, fuel2)
        }
    }
    strong_induction(f)
    forall(m: Nat) {
        if true_below(f, m) {
            forall(fuel2: Nat) {
                if Nat.1 < p and m <= fuel2 {
                    if m = Nat.0 {
                        digit_sum_fuel_at_zero(p, fuel2)
                        digit_sum_fuel_at_zero(p, m)
                        digit_sum_fuel(p, m, fuel2) = digit_sum(p, m)
                    }
                    if m != Nat.0 {
                        zero_or_suc(m)
                        let k: Nat satisfy { m = k.suc }
                        zero_or_suc(fuel2)
                        if fuel2 = Nat.0 {
                            only_zero_lte_zero(m)
                            false
                        }
                        let h: Nat satisfy { fuel2 = h.suc }
                        let q: Nat = m.div(p)
                        div_lt(m, p)
                        q < m
                        lt_imp_lte_suc(q, m)
                        lte_cancel_suc(q, k)
                        lte_cancel_suc(k, h)
                        k <= h
                        lte_trans(q, k, h)
                        suc_sub_one(k)
                        true_below_apply(f, m, q)
                        digit_sum_fuel_large_case(p, q, h)
                        Nat.1 < p and q <= h
                        digit_sum_fuel(p, q, h) = digit_sum(p, q)
                        digit_sum_fuel_large_case(p, q, k)
                        Nat.1 < p and q <= k
                        digit_sum_fuel(p, q, k) = digit_sum(p, q)
                        digit_sum_fuel_step(p, m, h)
                        digit_sum_fuel_step(p, m, k)
                        digit_sum_fuel(p, m, fuel2) = digit_sum(p, m)
                    }
                    digit_sum_fuel_large_case(p, m, fuel2)
                }
            }
            forall(fuel2: Nat) {
                digit_sum_fuel_large_case(p, m, fuel2)
            }
            f(m)
        }
    }
    f(n)
    digit_sum_fuel_large_case(p, n, fuel)
}

/// The digit sum recurrence for a nonzero number in a base greater than one.
theorem digit_sum_recurrence(p: Nat, n: Nat) {
    Nat.1 < p and n != Nat.0 implies
        digit_sum(p, n) = n.mod(p) + digit_sum(p, n.div(p))
} by {
    if Nat.1 < p and n != Nat.0 {
        zero_or_suc(n)
        if n = Nat.0 {
            false
        }
        let k: Nat satisfy { n = k.suc }
        let q: Nat = n.div(p)
        div_lt(n, p)
        q < n
        lt_imp_lte_suc(q, n)
        lte_cancel_suc(q, k)
        q <= k
        suc_sub_one(k)
        digit_sum_step(p, n)
        digit_sum_fuel_large(p, q, n - Nat.1)
        n.mod(p) + digit_sum_fuel(p, q, n - Nat.1) =
            n.mod(p) + digit_sum(p, q)
        digit_sum(p, n) = n.mod(p) + digit_sum(p, n.div(p))
    }
}

/// The base-`p` digit sum of zero is zero.
theorem digit_sum_zero(p: Nat) {
    digit_sum(p, Nat.0) = Nat.0
} by {
    digit_sum_fuel_at_zero(p, Nat.0)
}

/// Multiplication by the base appends a zero digit and preserves the digit sum.
theorem digit_sum_mul_base(p: Nat, n: Nat) {
    Nat.1 < p implies digit_sum(p, n * p) = digit_sum(p, n)
} by {
    if Nat.1 < p {
        p != Nat.0
        if n = Nat.0 {
            n * p = Nat.0
            digit_sum_zero(p)
            digit_sum(p, n * p) = digit_sum(p, n)
        } else {
            if n * p = Nat.0 {
                mul_to_zero(n, p)
                false
            }
            digit_sum_recurrence(p, n * p)
            mod_mul(p, n)
            div_mul(n, p)
            digit_sum(p, n * p) = digit_sum(p, n)
        }
        digit_sum(p, n * p) = digit_sum(p, n)
    }
}

/// Multiplication by two preserves the binary digit sum.
theorem digit_sum_two_mul(n: Nat) {
    digit_sum(Nat.2, n * Nat.2) = digit_sum(Nat.2, n)
} by {
    Nat.1 < Nat.2
    digit_sum_mul_base(Nat.2, n)
}

/// Doubling preserves the binary digit sum.
theorem digit_sum_two_double(n: Nat) {
    digit_sum(Nat.2, n + n) = digit_sum(Nat.2, n)
} by {
    digit_sum_two_mul(n)
}
