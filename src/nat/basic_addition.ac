from nat.nat_monoid import Nat
numerals Nat

/// Single-digit addition facts for natural numbers.
/// These facts can be used as building blocks for more complex arithmetic proofs.

theorem one_plus_two {
    1 + 2 = 3
}

theorem one_plus_three {
    1 + 3 = 4
}

theorem one_plus_four {
    1 + 4 = 5
}

theorem one_plus_five {
    1 + 5 = 6
}

theorem one_plus_six {
    1 + 6 = 7
}

theorem one_plus_seven {
    1 + 7 = 8
}

theorem one_plus_eight {
    1 + 8 = 9
}

theorem one_plus_nine {
    1 + 9 = 10
}

theorem one_plus_ten {
    1 + 10 = 11
}

theorem two_plus_one {
    2 + 1 = 3
}

theorem two_plus_two {
    2 + 2 = 4
}

theorem two_plus_three {
    2 + 3 = 5
}

theorem two_plus_four {
    2 + 4 = 6
}

theorem two_plus_five {
    2 + 5 = 7
}

theorem two_plus_six {
    2 + 6 = 8
}

theorem two_plus_seven {
    2 + 7 = 9
}

theorem two_plus_eight {
    2 + 8 = 10
}

theorem two_plus_nine {
    2 + 9 = 11
}

theorem two_plus_ten {
    2 + 10 = 12
}

theorem three_plus_one {
    3 + 1 = 4
}

theorem three_plus_two {
    3 + 2 = 5
}

theorem three_plus_three {
    3 + 3 = 6
}

theorem three_plus_four {
    3 + 4 = 7
}

theorem three_plus_five {
    3 + 5 = 8
}

theorem three_plus_six {
    3 + 6 = 9
}

theorem three_plus_seven {
    3 + 7 = 10
}

theorem three_plus_eight {
    3 + 8 = 11
}

theorem three_plus_nine {
    3 + 9 = 12
}

theorem three_plus_ten {
    3 + 10 = 13
}

theorem four_plus_one {
    4 + 1 = 5
}

theorem four_plus_two {
    4 + 2 = 6
}

theorem four_plus_three {
    4 + 3 = 7
}

theorem four_plus_four {
    4 + 4 = 8
}

theorem four_plus_five {
    4 + 5 = 9
}

theorem four_plus_six {
    4 + 6 = 10
}

theorem four_plus_seven {
    4 + 7 = 11
}

theorem four_plus_eight {
    4 + 8 = 12
}

theorem four_plus_nine {
    4 + 9 = 13
}

theorem four_plus_ten {
    4 + 10 = 14
}

theorem five_plus_one {
    5 + 1 = 6
}

theorem five_plus_two {
    5 + 2 = 7
}

theorem five_plus_three {
    5 + 3 = 8
}

theorem five_plus_four {
    5 + 4 = 9
}

theorem five_plus_five {
    5 + 5 = 10
}

theorem five_plus_six {
    5 + 6 = 11
}

theorem five_plus_seven {
    5 + 7 = 12
}

theorem five_plus_eight {
    5 + 8 = 13
}

theorem five_plus_nine {
    5 + 9 = 14
}

theorem five_plus_ten {
    5 + 10 = 15
}

theorem six_plus_one {
    6 + 1 = 7
}

theorem six_plus_two {
    6 + 2 = 8
}

theorem six_plus_three {
    6 + 3 = 9
}

theorem six_plus_four {
    6 + 4 = 10
}

theorem six_plus_five {
    6 + 5 = 11
}

theorem six_plus_six {
    6 + 6 = 12
}

theorem six_plus_seven {
    6 + 7 = 13
}

theorem six_plus_eight {
    6 + 8 = 14
}

theorem six_plus_nine {
    6 + 9 = 15
}

theorem six_plus_ten {
    6 + 10 = 16
}

theorem seven_plus_one {
    7 + 1 = 8
}

theorem seven_plus_two {
    7 + 2 = 9
}

theorem seven_plus_three {
    7 + 3 = 10
}

theorem seven_plus_four {
    7 + 4 = 11
}

theorem seven_plus_five {
    7 + 5 = 12
}

theorem seven_plus_six {
    7 + 6 = 13
}

theorem seven_plus_seven {
    7 + 7 = 14
}

theorem seven_plus_eight {
    7 + 8 = 15
}

theorem seven_plus_nine {
    7 + 9 = 16
}

theorem seven_plus_ten {
    7 + 10 = 17
}

theorem eight_plus_one {
    8 + 1 = 9
}

theorem eight_plus_two {
    8 + 2 = 10
}

theorem eight_plus_three {
    8 + 3 = 11
}

theorem eight_plus_four {
    8 + 4 = 12
}

theorem eight_plus_five {
    8 + 5 = 13
}

theorem eight_plus_six {
    8 + 6 = 14
}

theorem eight_plus_seven {
    8 + 7 = 15
}

theorem eight_plus_eight {
    8 + 8 = 16
}

theorem eight_plus_nine {
    8 + 9 = 17
}

theorem eight_plus_ten {
    8 + 10 = 18
}

theorem nine_plus_one {
    9 + 1 = 10
}

theorem nine_plus_two {
    9 + 2 = 11
}

theorem nine_plus_three {
    9 + 3 = 12
}

theorem nine_plus_four {
    9 + 4 = 13
}

theorem nine_plus_five {
    9 + 5 = 14
}

theorem nine_plus_six {
    9 + 6 = 15
}

theorem nine_plus_seven {
    9 + 7 = 16
}

theorem nine_plus_eight {
    9 + 8 = 17
}

theorem nine_plus_nine {
    9 + 9 = 18
}

theorem nine_plus_ten {
    9 + 10 = 19
}

theorem ten_plus_one {
    10 + 1 = 11
}

theorem ten_plus_two {
    10 + 2 = 12
}

theorem ten_plus_three {
    10 + 3 = 13
}

theorem ten_plus_four {
    10 + 4 = 14
}

theorem ten_plus_five {
    10 + 5 = 15
}

theorem ten_plus_six {
    10 + 6 = 16
}

theorem ten_plus_seven {
    10 + 7 = 17
}

theorem ten_plus_eight {
    10 + 8 = 18
}

theorem ten_plus_nine {
    10 + 9 = 19
}

theorem ten_plus_ten {
    10 + 10 = 20
}
