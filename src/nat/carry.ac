from nat.nat_base import Nat, add_cancels_right, add_zero_left, add_zero_right,
    add_assoc, add_comm, mul_cancel_left, mul_zero_left, small_mod
from nat.division import div_mod_decomp, div_mul
from nat.base_b_extra import div_of_decomp_eq, mod_of_decomp_eq, zero_div_by_nonzero
from nat.digit_sum import digit_sum, digit_sum_fuel, digit_sum_fuel_at_zero,
    digit_sum_fuel_step, digit_sum_recurrence, digit_sum_zero
numerals Nat

/// The lower-place total formed while adding two base-`p` digits and an
/// incoming carry.
define carry_total(p: Nat, a: Nat, b: Nat, c: Nat) -> Nat {
    a.mod(p) + b.mod(p) + c
}

/// The resulting base-`p` digit after adding two lower digits and a carry.
define carry_digit(p: Nat, a: Nat, b: Nat, c: Nat) -> Nat {
    carry_total(p, a, b, c).mod(p)
}

/// The carry sent to the next base-`p` digit.
define carry_out(p: Nat, a: Nat, b: Nat, c: Nat) -> Nat {
    carry_total(p, a, b, c).div(p)
}

/// The higher-place quotient after adding the lower digits of `a` and `b`
/// with incoming carry `c`.
define carry_high(p: Nat, a: Nat, b: Nat, c: Nat) -> Nat {
    a.div(p) + b.div(p) + carry_out(p, a, b, c)
}

/// A finite recursion counting carries while adding `a`, `b`, and an incoming
/// carry `c` in base `p`.
define carry_count_fuel(p: Nat, a: Nat, b: Nat, c: Nat, fuel: Nat) -> Nat {
    match fuel {
        Nat.zero {
            Nat.0
        }
        Nat.suc(k) {
            carry_out(p, a, b, c) +
                carry_count_fuel(p, a.div(p), b.div(p), carry_out(p, a, b, c), k)
        }
    }
}

/// The high state remaining after a finite number of carry steps.
define carry_high_fuel(p: Nat, a: Nat, b: Nat, c: Nat, fuel: Nat) -> Nat {
    match fuel {
        Nat.zero {
            a + b + c
        }
        Nat.suc(k) {
            carry_high_fuel(p, a.div(p), b.div(p), carry_out(p, a, b, c), k)
        }
    }
}

/// The sum of the emitted carry digits during a finite number of carry steps.
define carry_digit_sum_fuel(p: Nat, a: Nat, b: Nat, c: Nat, fuel: Nat) -> Nat {
    match fuel {
        Nat.zero {
            Nat.0
        }
        Nat.suc(k) {
            carry_digit(p, a, b, c) +
                carry_digit_sum_fuel(p, a.div(p), b.div(p), carry_out(p, a, b, c), k)
        }
    }
}

/// Repeated quotienting by `p`.
define iterated_div(p: Nat, n: Nat, fuel: Nat) -> Nat {
    match fuel {
        Nat.zero {
            n
        }
        Nat.suc(k) {
            iterated_div(p, n.div(p), k)
        }
    }
}

/// True if `count` has the digit-sum behavior of the carry count for
/// `a + b` in base `p`.
define is_addition_carry_count(p: Nat, a: Nat, b: Nat, count: Nat) -> Bool {
    p * count + digit_sum(p, a + b) =
        count + digit_sum(p, a) + digit_sum(p, b)
}

/// The lower-place total is symmetric in the two addends.
theorem carry_total_comm(p: Nat, a: Nat, b: Nat, c: Nat) {
    carry_total(p, a, b, c) = carry_total(p, b, a, c)
} by {
    a.mod(p) + b.mod(p) + c = b.mod(p) + a.mod(p) + c
}

/// The resulting digit is symmetric in the two addends.
theorem carry_digit_comm(p: Nat, a: Nat, b: Nat, c: Nat) {
    carry_digit(p, a, b, c) = carry_digit(p, b, a, c)
} by {
    carry_total_comm(p, a, b, c)
}

/// The outgoing carry is symmetric in the two addends.
theorem carry_out_comm(p: Nat, a: Nat, b: Nat, c: Nat) {
    carry_out(p, a, b, c) = carry_out(p, b, a, c)
} by {
    carry_total_comm(p, a, b, c)
}

/// The high part of the addition is symmetric in the two addends.
theorem carry_high_comm(p: Nat, a: Nat, b: Nat, c: Nat) {
    carry_high(p, a, b, c) = carry_high(p, b, a, c)
} by {
    carry_out_comm(p, a, b, c)
    a.div(p) + b.div(p) + carry_out(p, a, b, c) =
        b.div(p) + a.div(p) + carry_out(p, b, a, c)
}

/// The lower-place total is reconstructed from its outgoing carry and digit.
theorem carry_total_decomp(p: Nat, a: Nat, b: Nat, c: Nat) {
    carry_out(p, a, b, c) * p + carry_digit(p, a, b, c) =
        carry_total(p, a, b, c)
} by {
    div_mod_decomp(carry_total(p, a, b, c), p)
}

/// The lower-place total is the displayed sum of the lower residues.
theorem carry_total_eq(p: Nat, a: Nat, b: Nat, c: Nat) {
    carry_total(p, a, b, c) = a.mod(p) + b.mod(p) + c
}

/// The carry digit is the remainder of the lower-place total.
theorem carry_digit_eq(p: Nat, a: Nat, b: Nat, c: Nat) {
    carry_digit(p, a, b, c) = carry_total(p, a, b, c).mod(p)
}

/// The outgoing carry is the quotient of the lower-place total.
theorem carry_out_eq(p: Nat, a: Nat, b: Nat, c: Nat) {
    carry_out(p, a, b, c) = carry_total(p, a, b, c).div(p)
}

/// The higher-place quotient unfolds to the quotient digits and the outgoing
/// carry.
theorem carry_high_eq(p: Nat, a: Nat, b: Nat, c: Nat) {
    carry_high(p, a, b, c) = a.div(p) + b.div(p) + carry_out(p, a, b, c)
}

/// If the modulus is nonzero, the carry digit is below the base.
theorem carry_digit_lt_base(p: Nat, a: Nat, b: Nat, c: Nat) {
    p != Nat.0 implies carry_digit(p, a, b, c) < p
} by {
    if p != Nat.0 {
        carry_digit(p, a, b, c) < p
    }
}

/// A lower-place total smaller than the base produces no outgoing carry.
theorem carry_out_of_total_lt_base(p: Nat, a: Nat, b: Nat, c: Nat) {
    carry_total(p, a, b, c) < p implies carry_out(p, a, b, c) = Nat.0
} by {
    if carry_total(p, a, b, c) < p {
        div_of_decomp_eq(carry_total(p, a, b, c), Nat.0,
            carry_total(p, a, b, c), p)
        carry_total(p, a, b, c).div(p) = Nat.0
        carry_out(p, a, b, c) = Nat.0
    }
}

/// A lower-place total smaller than the base is itself the carry digit.
theorem carry_digit_of_total_lt_base(p: Nat, a: Nat, b: Nat, c: Nat) {
    carry_total(p, a, b, c) < p implies
        carry_digit(p, a, b, c) = carry_total(p, a, b, c)
} by {
    if carry_total(p, a, b, c) < p {
        mod_of_decomp_eq(carry_total(p, a, b, c), Nat.0,
            carry_total(p, a, b, c), p)
        carry_total(p, a, b, c).mod(p) = carry_total(p, a, b, c)
        carry_digit(p, a, b, c) = carry_total(p, a, b, c)
    }
}

/// Splitting both addends into quotient and remainder reconstructs the sum
/// using the carry digit and high part.
theorem carry_add_decomp(p: Nat, a: Nat, b: Nat, c: Nat) {
    carry_high(p, a, b, c) * p + carry_digit(p, a, b, c) =
        a + b + c
} by {
    div_mod_decomp(a, p)
    div_mod_decomp(b, p)

    carry_total_decomp(p, a, b, c)

    (a.div(p) + b.div(p) + carry_out(p, a, b, c)) * p =
        a.div(p) * p + b.div(p) * p + carry_out(p, a, b, c) * p
    carry_high(p, a, b, c) * p + carry_digit(p, a, b, c) =
        a.div(p) * p + b.div(p) * p +
            (carry_out(p, a, b, c) * p + carry_digit(p, a, b, c))
    a.div(p) * p + b.div(p) * p + (a.mod(p) + b.mod(p) + c) =
        (a.div(p) * p + a.mod(p)) + (b.div(p) * p + b.mod(p)) + c
}

/// The quotient of `a + b + c` is the high part of the carry decomposition.
theorem carry_add_div(p: Nat, a: Nat, b: Nat, c: Nat) {
    p != Nat.0 implies (a + b + c).div(p) = carry_high(p, a, b, c)
} by {
    if p != Nat.0 {
        carry_add_decomp(p, a, b, c)
        carry_digit_lt_base(p, a, b, c)
        div_of_decomp_eq(a + b + c, carry_high(p, a, b, c),
            carry_digit(p, a, b, c), p)
        (a + b + c).div(p) = carry_high(p, a, b, c)
    }
}

/// The remainder of `a + b + c` is the carry digit.
theorem carry_add_mod(p: Nat, a: Nat, b: Nat, c: Nat) {
    p != Nat.0 implies (a + b + c).mod(p) = carry_digit(p, a, b, c)
} by {
    if p != Nat.0 {
        carry_add_decomp(p, a, b, c)
        carry_digit_lt_base(p, a, b, c)
        mod_of_decomp_eq(a + b + c, carry_high(p, a, b, c),
            carry_digit(p, a, b, c), p)
        (a + b + c).mod(p) = carry_digit(p, a, b, c)
    }
}

/// The zero fuel count contains no carries.
theorem carry_count_fuel_zero(p: Nat, a: Nat, b: Nat, c: Nat) {
    carry_count_fuel(p, a, b, c, Nat.0) = Nat.0
}

/// One step of the fuelled carry-count recursion.
theorem carry_count_fuel_suc(p: Nat, a: Nat, b: Nat, c: Nat, fuel: Nat) {
    carry_count_fuel(p, a, b, c, fuel.suc) =
        carry_out(p, a, b, c) +
        carry_count_fuel(p, a.div(p), b.div(p), carry_out(p, a, b, c), fuel)
}

/// The zero fuel high state is the original sum.
theorem carry_high_fuel_zero(p: Nat, a: Nat, b: Nat, c: Nat) {
    carry_high_fuel(p, a, b, c, Nat.0) = a + b + c
}

/// One step of the high-state recursion.
theorem carry_high_fuel_suc(p: Nat, a: Nat, b: Nat, c: Nat, fuel: Nat) {
    carry_high_fuel(p, a, b, c, fuel.suc) =
        carry_high_fuel(p, a.div(p), b.div(p), carry_out(p, a, b, c), fuel)
}

/// The zero fuel emitted digit sum is zero.
theorem carry_digit_sum_fuel_zero(p: Nat, a: Nat, b: Nat, c: Nat) {
    carry_digit_sum_fuel(p, a, b, c, Nat.0) = Nat.0
}

/// One step of the emitted digit-sum recursion.
theorem carry_digit_sum_fuel_suc(p: Nat, a: Nat, b: Nat, c: Nat, fuel: Nat) {
    carry_digit_sum_fuel(p, a, b, c, fuel.suc) =
        carry_digit(p, a, b, c) +
        carry_digit_sum_fuel(p, a.div(p), b.div(p), carry_out(p, a, b, c), fuel)
}

/// Zero repeated divisions leave the number unchanged.
theorem iterated_div_zero(p: Nat, n: Nat) {
    iterated_div(p, n, Nat.0) = n
}

/// One repeated-division step removes the lowest base-`p` digit.
theorem iterated_div_suc(p: Nat, n: Nat, fuel: Nat) {
    iterated_div(p, n, fuel.suc) = iterated_div(p, n.div(p), fuel)
}

/// Repeated division of zero by a nonzero base remains zero.
theorem iterated_div_zero_value(p: Nat, fuel: Nat) {
    p != Nat.0 implies iterated_div(p, Nat.0, fuel) = Nat.0
} by {
    define pred(k: Nat) -> Bool {
        iterated_div(p, Nat.0, k) = Nat.0
    }

    pred(Nat.0)
    forall(k: Nat) {
        if pred(k) {
            if p != Nat.0 {
                iterated_div_suc(p, Nat.0, k)
                zero_div_by_nonzero(p)
                iterated_div(p, Nat.0, k.suc) = iterated_div(p, Nat.0, k)
                iterated_div(p, Nat.0, k.suc) = Nat.0
            }
            if p != Nat.0 {
                pred(k.suc)
            }
        }
    }
    if p != Nat.0 {
        pred(fuel)
        iterated_div(p, Nat.0, fuel) = Nat.0
    }
}

/// The fuelled carry count is symmetric in the two addends.
theorem carry_count_fuel_comm(p: Nat, a: Nat, b: Nat, c: Nat, fuel: Nat) {
    carry_count_fuel(p, a, b, c, fuel) =
        carry_count_fuel(p, b, a, c, fuel)
} by {
    define pred(k: Nat) -> Bool {
        forall(x: Nat, y: Nat, z: Nat) {
            carry_count_fuel(p, x, y, z, k) =
                carry_count_fuel(p, y, x, z, k)
        }
    }

    forall(x: Nat, y: Nat, z: Nat) {
        carry_count_fuel(p, y, x, z, Nat.0) = Nat.0
    }
    pred(Nat.0)
    forall(k: Nat) {
        if pred(k) {
            forall(x: Nat, y: Nat, z: Nat) {
                carry_count_fuel_suc(p, x, y, z, k)
                carry_count_fuel_suc(p, y, x, z, k)
                carry_out_comm(p, x, y, z)
                let h: Bool = carry_count_fuel(p, x.div(p), y.div(p),
                    carry_out(p, x, y, z), k) =
                    carry_count_fuel(p, y.div(p), x.div(p),
                    carry_out(p, x, y, z), k)
                carry_count_fuel(p, x.div(p), y.div(p), carry_out(p, x, y, z), k) =
                    carry_count_fuel(p, y.div(p), x.div(p), carry_out(p, x, y, z), k)
                carry_count_fuel(p, x, y, z, k.suc) =
                    carry_count_fuel(p, y, x, z, k.suc)
            }
            forall(x: Nat, y: Nat, z: Nat) {
                carry_count_fuel(p, x, y, z, k.suc) =
                    carry_count_fuel(p, y, x, z, k.suc)
            }
            pred(k.suc)
        }
    }
    pred(fuel)
    let h: Bool = carry_count_fuel(p, a, b, c, fuel) =
        carry_count_fuel(p, b, a, c, fuel)
}

/// The finite high state is symmetric in the two addends.
theorem carry_high_fuel_comm(p: Nat, a: Nat, b: Nat, c: Nat, fuel: Nat) {
    carry_high_fuel(p, a, b, c, fuel) =
        carry_high_fuel(p, b, a, c, fuel)
} by {
    define pred(k: Nat) -> Bool {
        forall(x: Nat, y: Nat, z: Nat) {
            carry_high_fuel(p, x, y, z, k) =
                carry_high_fuel(p, y, x, z, k)
        }
    }

    forall(x: Nat, y: Nat, z: Nat) {
        x + y + z = y + x + z
        carry_high_fuel(p, x, y, z, Nat.0) =
            carry_high_fuel(p, y, x, z, Nat.0)
    }
    pred(Nat.0)

    forall(k: Nat) {
        if pred(k) {
            forall(x: Nat, y: Nat, z: Nat) {
                carry_high_fuel_suc(p, x, y, z, k)
                carry_high_fuel_suc(p, y, x, z, k)
                carry_out_comm(p, x, y, z)
                let h: Bool = carry_high_fuel(p, x.div(p), y.div(p),
                    carry_out(p, x, y, z), k) =
                    carry_high_fuel(p, y.div(p), x.div(p),
                    carry_out(p, x, y, z), k)
                h
                carry_high_fuel(p, x, y, z, k.suc) =
                    carry_high_fuel(p, y, x, z, k.suc)
            }
            forall(x: Nat, y: Nat, z: Nat) {
                carry_high_fuel(p, x, y, z, k.suc) =
                    carry_high_fuel(p, y, x, z, k.suc)
            }
            pred(k.suc)
        }
    }
    pred(fuel)
    let h: Bool = carry_high_fuel(p, a, b, c, fuel) =
        carry_high_fuel(p, b, a, c, fuel)
}

/// The finite emitted digit sum is symmetric in the two addends.
theorem carry_digit_sum_fuel_comm(p: Nat, a: Nat, b: Nat, c: Nat, fuel: Nat) {
    carry_digit_sum_fuel(p, a, b, c, fuel) =
        carry_digit_sum_fuel(p, b, a, c, fuel)
} by {
    define pred(k: Nat) -> Bool {
        forall(x: Nat, y: Nat, z: Nat) {
            carry_digit_sum_fuel(p, x, y, z, k) =
                carry_digit_sum_fuel(p, y, x, z, k)
        }
    }

    forall(x: Nat, y: Nat, z: Nat) {
        carry_digit_sum_fuel(p, y, x, z, Nat.0) = Nat.0
    }
    pred(Nat.0)

    forall(k: Nat) {
        if pred(k) {
            forall(x: Nat, y: Nat, z: Nat) {
                carry_digit_sum_fuel_suc(p, x, y, z, k)
                carry_digit_sum_fuel_suc(p, y, x, z, k)
                carry_digit_comm(p, x, y, z)
                carry_out_comm(p, x, y, z)
                let h: Bool = carry_digit_sum_fuel(p, x.div(p), y.div(p),
                    carry_out(p, x, y, z), k) =
                    carry_digit_sum_fuel(p, y.div(p), x.div(p),
                    carry_out(p, x, y, z), k)
                h
                carry_digit_sum_fuel(p, x, y, z, k.suc) =
                    carry_digit_sum_fuel(p, y, x, z, k.suc)
            }
            forall(x: Nat, y: Nat, z: Nat) {
                carry_digit_sum_fuel(p, x, y, z, k.suc) =
                    carry_digit_sum_fuel(p, y, x, z, k.suc)
            }
            pred(k.suc)
        }
    }
    pred(fuel)
    let h: Bool = carry_digit_sum_fuel(p, a, b, c, fuel) =
        carry_digit_sum_fuel(p, b, a, c, fuel)
}

/// The digit-sum recurrence also holds at zero in a base greater than one.
theorem digit_sum_mod_div(p: Nat, n: Nat) {
    Nat.1 < p implies digit_sum(p, n) = n.mod(p) + digit_sum(p, n.div(p))
} by {
    if Nat.1 < p {
        if n = Nat.0 {
            digit_sum_zero(p)
            small_mod(Nat.0, p)
            Nat.0.mod(p) = Nat.0
            zero_div_by_nonzero(p)
            Nat.0.div(p) = Nat.0
            digit_sum_zero(p)
            digit_sum(p, n) = n.mod(p) + digit_sum(p, n.div(p))
        } else {
            digit_sum_recurrence(p, n)
            digit_sum(p, n) = n.mod(p) + digit_sum(p, n.div(p))
        }
    }
}

/// One base-`p` digit of a sum is obtained from the local carry digit.
theorem digit_sum_carry_step(p: Nat, a: Nat, b: Nat, c: Nat) {
    Nat.1 < p implies
        digit_sum(p, a + b + c) =
            carry_digit(p, a, b, c) + digit_sum(p, carry_high(p, a, b, c))
} by {
    if Nat.1 < p {
        p != Nat.0
        digit_sum_mod_div(p, a + b + c)
        carry_add_mod(p, a, b, c)
        carry_add_div(p, a, b, c)
        digit_sum(p, a + b + c) =
            carry_digit(p, a, b, c) + digit_sum(p, carry_high(p, a, b, c))
    }
}

/// A finite carry expansion decomposes the digit sum of the total into the
/// emitted low digits and the remaining high state.
theorem digit_sum_carry_fuel_decomp(p: Nat, a: Nat, b: Nat, c: Nat, fuel: Nat) {
    Nat.1 < p implies
        digit_sum(p, a + b + c) =
            carry_digit_sum_fuel(p, a, b, c, fuel) +
            digit_sum(p, carry_high_fuel(p, a, b, c, fuel))
} by {
    define pred(k: Nat) -> Bool {
        forall(x: Nat, y: Nat, z: Nat) {
            Nat.1 < p implies
                digit_sum(p, x + y + z) =
                    carry_digit_sum_fuel(p, x, y, z, k) +
                    digit_sum(p, carry_high_fuel(p, x, y, z, k))
        }
    }

    forall(x: Nat, y: Nat, z: Nat) {
        if Nat.1 < p {
            carry_digit_sum_fuel_zero(p, x, y, z)
            carry_high_fuel_zero(p, x, y, z)
            digit_sum(p, x + y + z) =
                carry_digit_sum_fuel(p, x, y, z, Nat.0) +
                digit_sum(p, carry_high_fuel(p, x, y, z, Nat.0))
        }
    }
    pred(Nat.0)

    forall(k: Nat) {
        if pred(k) {
            forall(x: Nat, y: Nat, z: Nat) {
                if Nat.1 < p {
                    digit_sum_carry_step(p, x, y, z)
                    let h: Bool =
                        digit_sum(p, x.div(p) + y.div(p) + carry_out(p, x, y, z)) =
                            carry_digit_sum_fuel(p, x.div(p), y.div(p),
                                carry_out(p, x, y, z), k) +
                            digit_sum(p, carry_high_fuel(p, x.div(p), y.div(p),
                                carry_out(p, x, y, z), k))
                    h
                    digit_sum(p, carry_high(p, x, y, z)) =
                        carry_digit_sum_fuel(p, x.div(p), y.div(p),
                            carry_out(p, x, y, z), k) +
                        digit_sum(p, carry_high_fuel(p, x.div(p), y.div(p),
                            carry_out(p, x, y, z), k))
                    carry_digit_sum_fuel_suc(p, x, y, z, k)
                    carry_high_fuel_suc(p, x, y, z, k)
                    carry_digit(p, x, y, z) +
                        (carry_digit_sum_fuel(p, x.div(p), y.div(p),
                            carry_out(p, x, y, z), k) +
                        digit_sum(p, carry_high_fuel(p, x.div(p), y.div(p),
                            carry_out(p, x, y, z), k))) =
                        carry_digit_sum_fuel(p, x, y, z, k.suc) +
                        digit_sum(p, carry_high_fuel(p, x, y, z, k.suc))
                    digit_sum(p, x + y + z) =
                        carry_digit_sum_fuel(p, x, y, z, k.suc) +
                        digit_sum(p, carry_high_fuel(p, x, y, z, k.suc))
                }
            }
            forall(x: Nat, y: Nat, z: Nat) {
                if Nat.1 < p {
                    digit_sum(p, x + y + z) =
                        carry_digit_sum_fuel(p, x, y, z, k.suc) +
                        digit_sum(p, carry_high_fuel(p, x, y, z, k.suc))
                }
            }
            pred(k.suc)
        }
    }

    if Nat.1 < p {
        pred(fuel)
        let h: Bool =
            digit_sum(p, a + b + c) =
                carry_digit_sum_fuel(p, a, b, c, fuel) +
                digit_sum(p, carry_high_fuel(p, a, b, c, fuel))
        digit_sum(p, a + b + c) =
            carry_digit_sum_fuel(p, a, b, c, fuel) +
            digit_sum(p, carry_high_fuel(p, a, b, c, fuel))
    }
}

/// With a zero right addend, the finite high carry state is repeated division
/// of the left addend.
theorem carry_high_fuel_zero_right(p: Nat, a: Nat, fuel: Nat) {
    p != Nat.0 implies
        carry_high_fuel(p, a, Nat.0, Nat.0, fuel) = iterated_div(p, a, fuel)
} by {
    define pred(k: Nat) -> Bool {
        forall(x: Nat) {
            carry_high_fuel(p, x, Nat.0, Nat.0, k) = iterated_div(p, x, k)
        }
    }

    forall(x: Nat) {
        carry_high_fuel_zero(p, x, Nat.0, Nat.0)
        iterated_div_zero(p, x)
        carry_high_fuel(p, x, Nat.0, Nat.0, Nat.0) = iterated_div(p, x, Nat.0)
    }
    pred(Nat.0)

    forall(k: Nat) {
        if pred(k) {
            forall(x: Nat) {
                if p != Nat.0 {
                    carry_high_fuel_suc(p, x, Nat.0, Nat.0, k)
                    zero_div_by_nonzero(p)
                    Nat.0.div(p) = Nat.0
                    small_mod(Nat.0, p)
                    Nat.0.mod(p) = Nat.0
                    carry_total(p, x, Nat.0, Nat.0) = x.mod(p)
                    x.mod(p) < p
                    carry_out_of_total_lt_base(p, x, Nat.0, Nat.0)
                    carry_out(p, x, Nat.0, Nat.0) = Nat.0
                    let h: Bool =
                        carry_high_fuel(p, x.div(p), Nat.0, Nat.0, k) =
                            iterated_div(p, x.div(p), k)
                    h
                    iterated_div_suc(p, x, k)
                    carry_high_fuel(p, x, Nat.0, Nat.0, k.suc) =
                        iterated_div(p, x, k.suc)
                }
            }
            forall(x: Nat) {
                if p != Nat.0 {
                    carry_high_fuel(p, x, Nat.0, Nat.0, k.suc) =
                        iterated_div(p, x, k.suc)
                }
            }
            if p != Nat.0 {
                pred(k.suc)
            }
        }
    }

    if p != Nat.0 {
        pred(fuel)
        let h: Bool =
            carry_high_fuel(p, a, Nat.0, Nat.0, fuel) = iterated_div(p, a, fuel)
        carry_high_fuel(p, a, Nat.0, Nat.0, fuel) = iterated_div(p, a, fuel)
    }
}

/// With a zero left addend, the finite high carry state is repeated division
/// of the right addend.
theorem carry_high_fuel_zero_left(p: Nat, b: Nat, fuel: Nat) {
    p != Nat.0 implies
        carry_high_fuel(p, Nat.0, b, Nat.0, fuel) = iterated_div(p, b, fuel)
} by {
    if p != Nat.0 {
        carry_high_fuel_comm(p, Nat.0, b, Nat.0, fuel)
        carry_high_fuel_zero_right(p, b, fuel)
        carry_high_fuel(p, Nat.0, b, Nat.0, fuel) = iterated_div(p, b, fuel)
    }
}

/// One zero-addend carry digit-sum step agrees with the ordinary digit-sum step.
theorem carry_digit_sum_fuel_zero_right_step(p: Nat, a: Nat, k: Nat) {
    p != Nat.0 and forall(x: Nat) {
        carry_digit_sum_fuel(p, x, Nat.0, Nat.0, k) =
            digit_sum_fuel(p, x, k)
    } implies
        carry_digit_sum_fuel(p, a, Nat.0, Nat.0, k.suc) =
            digit_sum_fuel(p, a, k.suc)
} by {
    if p != Nat.0 and forall(x: Nat) {
        carry_digit_sum_fuel(p, x, Nat.0, Nat.0, k) =
            digit_sum_fuel(p, x, k)
    } {
        carry_digit_sum_fuel_suc(p, a, Nat.0, Nat.0, k)
        small_mod(Nat.0, p)
        Nat.0.mod(p) = Nat.0
        carry_total(p, a, Nat.0, Nat.0) = a.mod(p)
        a.mod(p) < p
        carry_digit_of_total_lt_base(p, a, Nat.0, Nat.0)
        carry_out_of_total_lt_base(p, a, Nat.0, Nat.0)
        carry_out(p, a, Nat.0, Nat.0) = Nat.0
        zero_div_by_nonzero(p)
        let h: Bool =
            carry_digit_sum_fuel(p, a.div(p), Nat.0, Nat.0, k) =
                digit_sum_fuel(p, a.div(p), k)
        carry_digit_sum_fuel(p, a, Nat.0, Nat.0, k.suc) =
            a.mod(p) + digit_sum_fuel(p, a.div(p), k)
        if a = Nat.0 {
            digit_sum_fuel_at_zero(p, k.suc)
            small_mod(Nat.0, p)
            zero_div_by_nonzero(p)
            digit_sum_fuel_at_zero(p, k)
            carry_digit_sum_fuel(p, a, Nat.0, Nat.0, k.suc) =
                digit_sum_fuel(p, a, k.suc)
        }
        if a != Nat.0 {
            digit_sum_fuel_step(p, a, k)
            carry_digit_sum_fuel(p, a, Nat.0, Nat.0, k.suc) =
                digit_sum_fuel(p, a, k.suc)
        }
        carry_digit_sum_fuel(p, a, Nat.0, Nat.0, k.suc) =
            digit_sum_fuel(p, a, k.suc)
    }
}

/// With a zero right addend, the emitted carry digits are the ordinary
/// explicit-fuel digit sum of the left addend.
theorem carry_digit_sum_fuel_zero_right(p: Nat, a: Nat, fuel: Nat) {
    p != Nat.0 implies
        carry_digit_sum_fuel(p, a, Nat.0, Nat.0, fuel) =
        digit_sum_fuel(p, a, fuel)
} by {
    if p != Nat.0 {
        define pred(k: Nat) -> Bool {
            forall(x: Nat) {
                carry_digit_sum_fuel(p, x, Nat.0, Nat.0, k) =
                    digit_sum_fuel(p, x, k)
            }
        }

        forall(x: Nat) {
            carry_digit_sum_fuel_zero(p, x, Nat.0, Nat.0)
            digit_sum_fuel(p, x, Nat.0) = Nat.0
            carry_digit_sum_fuel(p, x, Nat.0, Nat.0, Nat.0) =
                digit_sum_fuel(p, x, Nat.0)
        }
        pred(Nat.0)

        forall(k: Nat) {
            if pred(k) {
                forall(x: Nat) {
                    carry_digit_sum_fuel_zero_right_step(p, x, k)
                    carry_digit_sum_fuel(p, x, Nat.0, Nat.0, k.suc) =
                        digit_sum_fuel(p, x, k.suc)
                }
                pred(k.suc)
            }
        }

        pred(fuel)
        let h: Bool =
            carry_digit_sum_fuel(p, a, Nat.0, Nat.0, fuel) =
                digit_sum_fuel(p, a, fuel)
        carry_digit_sum_fuel(p, a, Nat.0, Nat.0, fuel) =
            digit_sum_fuel(p, a, fuel)
    }
}

/// With a zero left addend, the emitted carry digits are the ordinary
/// explicit-fuel digit sum of the right addend.
theorem carry_digit_sum_fuel_zero_left(p: Nat, b: Nat, fuel: Nat) {
    p != Nat.0 implies
        carry_digit_sum_fuel(p, Nat.0, b, Nat.0, fuel) =
        digit_sum_fuel(p, b, fuel)
} by {
    if p != Nat.0 {
        carry_digit_sum_fuel_comm(p, Nat.0, b, Nat.0, fuel)
        carry_digit_sum_fuel_zero_right(p, b, fuel)
        carry_digit_sum_fuel(p, Nat.0, b, Nat.0, fuel) =
            digit_sum_fuel(p, b, fuel)
    }
}

/// With no right addend and no incoming carry, the lower-place total is the
/// lower digit of the left addend.
theorem carry_total_zero_right(p: Nat, a: Nat) {
    carry_total(p, a, Nat.0, Nat.0) = a.mod(p)
} by {
    small_mod(Nat.0, p)
    Nat.0.mod(p) = Nat.0
}

/// With no left addend and no incoming carry, the lower-place total is the
/// lower digit of the right addend.
theorem carry_total_zero_left(p: Nat, b: Nat) {
    carry_total(p, Nat.0, b, Nat.0) = b.mod(p)
} by {
    small_mod(Nat.0, p)
}

/// Adding zero on the right produces no outgoing carry from any digit.
theorem carry_out_zero_right(p: Nat, a: Nat) {
    p != Nat.0 implies carry_out(p, a, Nat.0, Nat.0) = Nat.0
} by {
    if p != Nat.0 {
        carry_total_zero_right(p, a)
        a.mod(p) < p
        carry_out_of_total_lt_base(p, a, Nat.0, Nat.0)
        carry_out(p, a, Nat.0, Nat.0) = Nat.0
    }
}

/// Adding zero on the left produces no outgoing carry from any digit.
theorem carry_out_zero_left(p: Nat, b: Nat) {
    p != Nat.0 implies carry_out(p, Nat.0, b, Nat.0) = Nat.0
} by {
    if p != Nat.0 {
        carry_out_comm(p, Nat.0, b, Nat.0)
        carry_out_zero_right(p, b)
        carry_out(p, Nat.0, b, Nat.0) = Nat.0
    }
}

/// Adding zero on the right leaves the carry digit equal to the lower digit of
/// the left addend.
theorem carry_digit_zero_right(p: Nat, a: Nat) {
    p != Nat.0 implies carry_digit(p, a, Nat.0, Nat.0) = a.mod(p)
} by {
    if p != Nat.0 {
        carry_total_zero_right(p, a)
        a.mod(p) < p
        carry_digit_of_total_lt_base(p, a, Nat.0, Nat.0)
        carry_digit(p, a, Nat.0, Nat.0) = a.mod(p)
    }
}

/// Adding zero on the left leaves the carry digit equal to the lower digit of
/// the right addend.
theorem carry_digit_zero_left(p: Nat, b: Nat) {
    p != Nat.0 implies carry_digit(p, Nat.0, b, Nat.0) = b.mod(p)
} by {
    if p != Nat.0 {
        carry_digit_comm(p, Nat.0, b, Nat.0)
        carry_digit_zero_right(p, b)
        carry_digit(p, Nat.0, b, Nat.0) = b.mod(p)
    }
}

/// Adding zero on the right leaves the higher quotient equal to the quotient
/// of the left addend.
theorem carry_high_zero_right(p: Nat, a: Nat) {
    p != Nat.0 implies carry_high(p, a, Nat.0, Nat.0) = a.div(p)
} by {
    if p != Nat.0 {
        zero_div_by_nonzero(p)
        Nat.0.div(p) = Nat.0
        carry_out_zero_right(p, a)
        carry_out(p, a, Nat.0, Nat.0) = Nat.0
        carry_high(p, a, Nat.0, Nat.0) = a.div(p)
    }
}

/// Adding zero on the left leaves the higher quotient equal to the quotient
/// of the right addend.
theorem carry_high_zero_left(p: Nat, b: Nat) {
    p != Nat.0 implies carry_high(p, Nat.0, b, Nat.0) = b.div(p)
} by {
    if p != Nat.0 {
        carry_high_comm(p, Nat.0, b, Nat.0)
        carry_high_zero_right(p, b)
        carry_high(p, Nat.0, b, Nat.0) = b.div(p)
    }
}

/// Adding zero on the right gives zero carries for every finite recursion
/// budget.
theorem carry_count_fuel_zero_right(p: Nat, a: Nat, fuel: Nat) {
    p != Nat.0 implies carry_count_fuel(p, a, Nat.0, Nat.0, fuel) = Nat.0
} by {
    define pred(k: Nat) -> Bool {
        forall(x: Nat) {
            carry_count_fuel(p, x, Nat.0, Nat.0, k) = Nat.0
        }
    }

    forall(x: Nat) {
        carry_count_fuel(p, x, Nat.0, Nat.0, Nat.0) = Nat.0
    }
    pred(Nat.0)

    forall(k: Nat) {
        if pred(k) {
            forall(x: Nat) {
                if p != Nat.0 {
                    carry_count_fuel_suc(p, x, Nat.0, Nat.0, k)
                    carry_out_zero_right(p, x)
                    zero_div_by_nonzero(p)
                    Nat.0.div(p) = Nat.0
                    let h: Bool =
                        carry_count_fuel(p, x.div(p), Nat.0, Nat.0, k) = Nat.0
                    h
                    carry_count_fuel(p, x, Nat.0, Nat.0, k.suc) = Nat.0
                }
            }
            forall(x: Nat) {
                if p != Nat.0 {
                    carry_count_fuel(p, x, Nat.0, Nat.0, k.suc) = Nat.0
                }
            }
            if p != Nat.0 {
                pred(k.suc)
            }
        }
    }

    if p != Nat.0 {
        pred(fuel)
        let h: Bool = carry_count_fuel(p, a, Nat.0, Nat.0, fuel) = Nat.0
        carry_count_fuel(p, a, Nat.0, Nat.0, fuel) = Nat.0
    }
}

/// Adding zero on the left gives zero carries for every finite recursion
/// budget.
theorem carry_count_fuel_zero_left(p: Nat, b: Nat, fuel: Nat) {
    p != Nat.0 implies carry_count_fuel(p, Nat.0, b, Nat.0, fuel) = Nat.0
} by {
    if p != Nat.0 {
        carry_count_fuel_comm(p, Nat.0, b, Nat.0, fuel)
        carry_count_fuel_zero_right(p, b, fuel)
        carry_count_fuel(p, Nat.0, b, Nat.0, fuel) = Nat.0
    }
}

/// Adding zero on the right has carry count zero.
theorem zero_is_addition_carry_count_right(p: Nat, a: Nat) {
    Nat.1 < p implies is_addition_carry_count(p, a, Nat.0, Nat.0)
} by {
    if Nat.1 < p {
        digit_sum_zero(p)
        is_addition_carry_count(p, a, Nat.0, Nat.0)
    }
}

/// Adding zero on the left has carry count zero.
theorem zero_is_addition_carry_count_left(p: Nat, b: Nat) {
    Nat.1 < p implies is_addition_carry_count(p, Nat.0, b, Nat.0)
} by {
    if Nat.1 < p {
        zero_is_addition_carry_count_right(p, b)
        digit_sum_zero(p)
        is_addition_carry_count(p, Nat.0, b, Nat.0)
    }
}

/// The carry-count predicate is symmetric in the two addends.
theorem is_addition_carry_count_comm(p: Nat, a: Nat, b: Nat, count: Nat) {
    is_addition_carry_count(p, a, b, count) implies
        is_addition_carry_count(p, b, a, count)
} by {
    if is_addition_carry_count(p, a, b, count) {
        let sa: Nat = digit_sum(p, a)
        let sb: Nat = digit_sum(p, b)
        add_comm(sa, sb)
        add_assoc(count, sa, sb)
        add_assoc(count, sb, sa)
        count + digit_sum(p, a) + digit_sum(p, b) =
            count + digit_sum(p, b) + digit_sum(p, a)
        p * count + digit_sum(p, a + b) =
            count + digit_sum(p, b) + digit_sum(p, a)
        p * count + digit_sum(p, b + a) =
            count + digit_sum(p, b) + digit_sum(p, a)
        is_addition_carry_count(p, b, a, count)
    }
}
