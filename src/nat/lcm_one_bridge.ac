from nat.gcd import Nat, gcd_one_left, gcd_one_right
from nat.lcm import gcd_mul_lcm
from nat.nat_base import mul_one_left, mul_one_right
numerals Nat

/// The least common multiple of a natural number and one is the number itself.
theorem nat_lcm_one_right(a: Nat) {
    a.lcm(Nat.1) = a
} by {
    gcd_one_right(a)
    a.gcd(Nat.1) = Nat.1
    gcd_mul_lcm(a, Nat.1)
    a.gcd(Nat.1) * a.lcm(Nat.1) = a * Nat.1
    Nat.1 * a.lcm(Nat.1) = a * Nat.1
    mul_one_left(a.lcm(Nat.1))
    Nat.1 * a.lcm(Nat.1) = a.lcm(Nat.1)
    mul_one_right(a)
    a * Nat.1 = a
    a.lcm(Nat.1) = a
}

/// The least common multiple of one and a natural number is the number itself.
theorem nat_lcm_one_left(a: Nat) {
    Nat.1.lcm(a) = a
} by {
    gcd_one_left(a)
    Nat.1.gcd(a) = Nat.1
    gcd_mul_lcm(Nat.1, a)
    Nat.1.gcd(a) * Nat.1.lcm(a) = Nat.1 * a
    Nat.1 * Nat.1.lcm(a) = Nat.1 * a
    mul_one_left(Nat.1.lcm(a))
    Nat.1 * Nat.1.lcm(a) = Nat.1.lcm(a)
    mul_one_left(a)
    Nat.1 * a = a
    Nat.1.lcm(a) = a
}

/// Both lcm-with-one identities, packaged as a conjunction.
theorem nat_lcm_one_pair(a: Nat) {
    Nat.1.lcm(a) = a and a.lcm(Nat.1) = a
} by {
    nat_lcm_one_left(a)
    nat_lcm_one_right(a)
}
