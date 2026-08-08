from nat.digit_sum import Nat, digit_sum, digit_sum_fuel, digit_sum_zero,
    digit_sum_fuel_at_zero, digit_sum_fuel_step, digit_sum_recurrence,
    digit_sum_mul_base, zero_or_suc, lt_or_lte, lte_trans, lt_imp_lte_suc,
    lte_cancel_suc, only_zero_lte_zero, mul_to_zero
from nat.nat_base import lt_add_suc
from nat.basic_addition import one_plus_nine, two_plus_eight, three_plus_seven,
    four_plus_six, five_plus_five
from nat.basic_multiplication import nat_add_12_1, nat_add_2_1, nat_mul_2_1,
    nat_mul_2_3, nat_mul_2_6
from nat.base_b_extra import div_of_decomp_eq, mod_of_decomp_eq, mod_one, digit_sum_small
numerals Nat

/// The remainder of a base multiple plus a small digit is that digit.
theorem mod_base_mul_add_digit(p: Nat, n: Nat, d: Nat) {
    d < p implies (p * n + d).mod(p) = d
} by {
    if d < p {
        p * n = n * p
        p * n + d = n * p + d
        mod_of_decomp_eq(p * n + d, n, d, p)
        (p * n + d).mod(p) = d
    }
}

/// The quotient of a base multiple plus a small digit is the multiplier.
theorem div_base_mul_add_digit(p: Nat, n: Nat, d: Nat) {
    d < p implies (p * n + d).div(p) = n
} by {
    if d < p {
        p * n = n * p
        p * n + d = n * p + d
        div_of_decomp_eq(p * n + d, n, d, p)
        (p * n + d).div(p) = n
    }
}

/// Left-multiplication by the base appends a zero digit and preserves digit sum.
theorem digit_sum_base_mul(p: Nat, n: Nat) {
    Nat.1 < p implies digit_sum(p, p * n) = digit_sum(p, n)
} by {
    if Nat.1 < p {
        p * n = n * p
        digit_sum_mul_base(p, n)
        digit_sum(p, n * p) = digit_sum(p, n)
        digit_sum(p, p * n) = digit_sum(p, n)
    }
}

/// Any nonzero natural is at least one.
theorem nat_one_lte_of_nonzero(n: Nat) {
    n != Nat.0 implies Nat.1 <= n
} by {
    if n != Nat.0 {
        zero_or_suc(n)
        let k: Nat satisfy { n = k.suc }
        Nat.1 + k = k.suc
        Nat.1 <= n
    }
}

/// A natural is less than its successor.
theorem nat_lt_suc(n: Nat) {
    n < n.suc
} by {
    n <= n.suc
    n != n.suc
    n < n.suc
}

/// If a nonzero natural is below `p`, then `p` is greater than one.
theorem nat_one_lt_of_nonzero_lt(n: Nat, p: Nat) {
    n != Nat.0 and n < p implies Nat.1 < p
} by {
    if n != Nat.0 and n < p {
        nat_one_lte_of_nonzero(n)
        Nat.1 <= n
        lt_imp_lte_suc(n, p)
        n.suc <= p
        if p <= Nat.1 {
            lte_trans(n.suc, p, Nat.1)
            n.suc <= Nat.1
            lte_cancel_suc(n, Nat.0)
            n <= Nat.0
            only_zero_lte_zero(n)
            false
        }
        not p <= Nat.1
        lt_or_lte(Nat.1, p)
        Nat.1 < p
    }
}

/// Numbers below a nonzero base have digit sum equal to themselves. This wraps
/// the existing `digit_sum_small` while discharging the nonzero-small cases.
theorem digit_sum_lt_base(p: Nat, n: Nat) {
    n < p implies digit_sum(p, n) = n
} by {
    if n < p {
        if n = Nat.0 {
            digit_sum_zero(p)
            digit_sum(p, n) = n
        }
        if n != Nat.0 {
            nat_one_lt_of_nonzero_lt(n, p)
            Nat.1 < p
            digit_sum_small(p, n)
            digit_sum(p, n) = n
        }
        digit_sum(p, n) = n
    }
}

/// A nonzero natural that is not greater than one is one.
theorem nat_nonzero_not_gt_one_imp_one(p: Nat) {
    p != Nat.0 and not Nat.1 < p implies p = Nat.1
} by {
    if p != Nat.0 and not Nat.1 < p {
        zero_or_suc(p)
        let q: Nat satisfy { p = q.suc }
        if q = Nat.0 {
            p = Nat.1
        }
        if q != Nat.0 {
            nat_lt_suc(q)
            q < q.suc
            nat_one_lt_of_nonzero_lt(q, q.suc)
            Nat.1 < q.suc
            Nat.1 < p
            false
        }
    }
}

/// If a sum is zero, both summands are zero.
theorem nat_add_to_zero(a: Nat, b: Nat) {
    a + b = Nat.0 implies a = Nat.0 and b = Nat.0
} by {
    if a + b = Nat.0 {
        a = Nat.0
        b = Nat.0
    }
}

/// In base one the fuelled digit sum is always zero.
theorem digit_sum_fuel_base_one(fuel: Nat, n: Nat) {
    digit_sum_fuel(Nat.1, n, fuel) = Nat.0
} by {
    define pred(k: Nat) -> Bool {
        forall(x: Nat) { digit_sum_fuel(Nat.1, x, k) = Nat.0 }
    }
    forall(x: Nat) {
        digit_sum_fuel(Nat.1, x, Nat.0) = Nat.0
    }
    pred(Nat.0)
    forall(k: Nat) {
        if pred(k) {
            forall(x: Nat) {
                if x = Nat.0 {
                    digit_sum_fuel_at_zero(Nat.1, k.suc)
                    digit_sum_fuel(Nat.1, x, k.suc) = Nat.0
                }
                if x != Nat.0 {
                    digit_sum_fuel_step(Nat.1, x, k)
                    digit_sum_fuel(Nat.1, x, k.suc) =
                        x.mod(Nat.1) + digit_sum_fuel(Nat.1, x.div(Nat.1), k)
                    mod_one(x)
                    x.mod(Nat.1) = Nat.0
                    pred(k) = forall(y: Nat) { digit_sum_fuel(Nat.1, y, k) = Nat.0 }
                    let h: Bool = digit_sum_fuel(Nat.1, x.div(Nat.1), k) = Nat.0
                    h
                    digit_sum_fuel(Nat.1, x.div(Nat.1), k) = Nat.0
                    digit_sum_fuel(Nat.1, x, k.suc) = Nat.0 + Nat.0
                    digit_sum_fuel(Nat.1, x, k.suc) = Nat.0
                }
            }
            pred(k.suc)
        }
    }
    pred(fuel)
    pred(fuel) = forall(x: Nat) { digit_sum_fuel(Nat.1, x, fuel) = Nat.0 }
    let h: Bool = digit_sum_fuel(Nat.1, n, fuel) = Nat.0
    h
    digit_sum_fuel(Nat.1, n, fuel) = Nat.0
}

/// In base one the digit sum is always zero.
theorem digit_sum_base_one(n: Nat) {
    digit_sum(Nat.1, n) = Nat.0
} by {
    digit_sum(Nat.1, n) = digit_sum_fuel(Nat.1, n, n)
    digit_sum_fuel_base_one(n, n)
    digit_sum(Nat.1, n) = Nat.0
}

/// Appending a digit `d` below a base greater than one updates the digit sum by
/// adding that digit.
theorem digit_sum_base_mul_add_digit_gt_one(p: Nat, n: Nat, d: Nat) {
    d < p and Nat.1 < p implies digit_sum(p, p * n + d) = digit_sum(p, n) + d
} by {
    if d < p and Nat.1 < p {
        mod_base_mul_add_digit(p, n, d)
        (p * n + d).mod(p) = d
        div_base_mul_add_digit(p, n, d)
        (p * n + d).div(p) = n
        if p * n + d = Nat.0 {
            nat_add_to_zero(p * n, d)
            p * n = Nat.0
            d = Nat.0
            p != Nat.0
            mul_to_zero(p, n)
            n = Nat.0
            digit_sum_zero(p)
            digit_sum(p, p * n + d) = Nat.0
            digit_sum(p, n) = Nat.0
            digit_sum(p, n) + d = Nat.0
            digit_sum(p, p * n + d) = digit_sum(p, n) + d
        }
        if p * n + d != Nat.0 {
            digit_sum_recurrence(p, p * n + d)
            digit_sum(p, p * n + d) =
                (p * n + d).mod(p) + digit_sum(p, (p * n + d).div(p))
            digit_sum(p, p * n + d) = d + digit_sum(p, n)
            d + digit_sum(p, n) = digit_sum(p, n) + d
            digit_sum(p, p * n + d) = digit_sum(p, n) + d
        }
        digit_sum(p, p * n + d) = digit_sum(p, n) + d
    }
}

/// Appending a digit `d` below a nonzero base updates the digit sum by adding
/// that digit.
theorem digit_sum_base_mul_add_digit(p: Nat, n: Nat, d: Nat) {
    d < p and p != Nat.0 implies digit_sum(p, p * n + d) = digit_sum(p, n) + d
} by {
    if d < p and p != Nat.0 {
        if Nat.1 < p {
            digit_sum_base_mul_add_digit_gt_one(p, n, d)
            digit_sum(p, p * n + d) = digit_sum(p, n) + d
        }
        if not Nat.1 < p {
            nat_nonzero_not_gt_one_imp_one(p)
            p = Nat.1
            d < Nat.1
            lt_imp_lte_suc(d, Nat.1)
            d.suc <= Nat.1
            lte_cancel_suc(d, Nat.0)
            d <= Nat.0
            only_zero_lte_zero(d)
            d = Nat.0
            p * n = Nat.1 * n
            Nat.1 * n = n
            p * n + d = n + Nat.0
            p * n + d = n
            digit_sum_base_one(n)
            digit_sum(Nat.1, n) = Nat.0
            digit_sum(p, p * n + d) = Nat.0
            digit_sum(p, n) = Nat.0
            digit_sum(p, n) + d = Nat.0
            digit_sum(p, p * n + d) = digit_sum(p, n) + d
        }
        digit_sum(p, p * n + d) = digit_sum(p, n) + d
    }
}

/// The digit one is less than ten.
theorem digit_one_lt_ten {
    Nat.1 < Nat.10
} by {
    lt_add_suc(Nat.1, Nat.8)
    Nat.1 < Nat.1 + Nat.9
    one_plus_nine
    Nat.1 + Nat.9 = Nat.10
    Nat.1 < Nat.10
}

/// The digit two is less than ten.
theorem digit_two_lt_ten {
    Nat.2 < Nat.10
} by {
    lt_add_suc(Nat.2, Nat.7)
    Nat.2 < Nat.2 + Nat.8
    two_plus_eight
    Nat.2 + Nat.8 = Nat.10
    Nat.2 < Nat.10
}

/// The digit three is less than ten.
theorem digit_three_lt_ten {
    Nat.3 < Nat.10
} by {
    lt_add_suc(Nat.3, Nat.6)
    Nat.3 < Nat.3 + Nat.7
    three_plus_seven
    Nat.3 + Nat.7 = Nat.10
    Nat.3 < Nat.10
}

/// The digit four is less than ten.
theorem digit_four_lt_ten {
    Nat.4 < Nat.10
} by {
    lt_add_suc(Nat.4, Nat.5)
    Nat.4 < Nat.4 + Nat.6
    four_plus_six
    Nat.4 + Nat.6 = Nat.10
    Nat.4 < Nat.10
}

/// The digit five is less than ten.
theorem digit_five_lt_ten {
    Nat.5 < Nat.10
} by {
    lt_add_suc(Nat.5, Nat.4)
    Nat.5 < Nat.5 + Nat.5
    five_plus_five
    Nat.5 + Nat.5 = Nat.10
    Nat.5 < Nat.10
}

/// The base-ten digit sum of `12345` is `15`.
theorem digit_sum_ten_12345 {
    digit_sum(Nat.10, Nat.12345) = Nat.15
} by {
    digit_five_lt_ten
    Nat.5 < Nat.10
    digit_sum_base_mul_add_digit(Nat.10, Nat.1234, Nat.5)
    digit_sum(Nat.10, Nat.10 * Nat.1234 + Nat.5) = digit_sum(Nat.10, Nat.1234) + Nat.5
    digit_sum(Nat.10, Nat.12345) = digit_sum(Nat.10, Nat.1234) + Nat.5
    digit_four_lt_ten
    Nat.4 < Nat.10
    digit_sum_base_mul_add_digit(Nat.10, Nat.123, Nat.4)
    digit_sum(Nat.10, Nat.1234) = digit_sum(Nat.10, Nat.123) + Nat.4
    digit_three_lt_ten
    Nat.3 < Nat.10
    digit_sum_base_mul_add_digit(Nat.10, Nat.12, Nat.3)
    digit_sum(Nat.10, Nat.123) = digit_sum(Nat.10, Nat.12) + Nat.3
    digit_two_lt_ten
    Nat.2 < Nat.10
    digit_sum_base_mul_add_digit(Nat.10, Nat.1, Nat.2)
    digit_sum(Nat.10, Nat.12) = digit_sum(Nat.10, Nat.1) + Nat.2
    digit_one_lt_ten
    Nat.1 < Nat.10
    digit_sum_lt_base(Nat.10, Nat.1)
    digit_sum(Nat.10, Nat.1) = Nat.1
    digit_sum(Nat.10, Nat.12345) = Nat.15
}

/// The digit zero is less than two.
theorem digit_zero_lt_two {
    Nat.0 < Nat.2
} by {
    lt_add_suc(Nat.0, Nat.1)
    Nat.0 < Nat.0 + Nat.2
    Nat.0 < Nat.2
}

/// The digit one is less than two.
theorem digit_one_lt_two {
    Nat.1 < Nat.2
} by {
    lt_add_suc(Nat.1, Nat.0)
    Nat.1 < Nat.1 + Nat.1
    Nat.1 + Nat.1 = Nat.2
    Nat.1 < Nat.2
}

/// The binary digit sum of `13` (`1101` in base two) is `3`.
theorem digit_sum_two_13 {
    digit_sum(Nat.2, Nat.13) = Nat.3
} by {
    digit_one_lt_two
    Nat.1 < Nat.2
    Nat.2 * Nat.6 = Nat.12
    Nat.2 * Nat.6 + Nat.1 = Nat.12 + Nat.1
    nat_add_12_1
    Nat.12 + Nat.1 = Nat.13
    Nat.2 * Nat.6 + Nat.1 = Nat.13
    digit_sum_base_mul_add_digit(Nat.2, Nat.6, Nat.1)
    digit_sum(Nat.2, Nat.2 * Nat.6 + Nat.1) = digit_sum(Nat.2, Nat.6) + Nat.1
    digit_sum(Nat.2, Nat.13) = digit_sum(Nat.2, Nat.6) + Nat.1
    digit_zero_lt_two
    Nat.0 < Nat.2
    Nat.2 * Nat.3 = Nat.6
    Nat.2 * Nat.3 + Nat.0 = Nat.6 + Nat.0
    Nat.6 + Nat.0 = Nat.6
    Nat.2 * Nat.3 + Nat.0 = Nat.6
    digit_sum_base_mul_add_digit(Nat.2, Nat.3, Nat.0)
    digit_sum(Nat.2, Nat.2 * Nat.3 + Nat.0) = digit_sum(Nat.2, Nat.3) + Nat.0
    digit_sum(Nat.2, Nat.6) = digit_sum(Nat.2, Nat.3) + Nat.0
    digit_one_lt_two
    Nat.1 < Nat.2
    Nat.2 * Nat.1 = Nat.2
    Nat.2 * Nat.1 + Nat.1 = Nat.2 + Nat.1
    nat_add_2_1
    Nat.2 + Nat.1 = Nat.3
    Nat.2 * Nat.1 + Nat.1 = Nat.3
    digit_sum_base_mul_add_digit(Nat.2, Nat.1, Nat.1)
    digit_sum(Nat.2, Nat.2 * Nat.1 + Nat.1) = digit_sum(Nat.2, Nat.1) + Nat.1
    digit_sum(Nat.2, Nat.3) = digit_sum(Nat.2, Nat.1) + Nat.1
    digit_one_lt_two
    Nat.1 < Nat.2
    digit_sum_lt_base(Nat.2, Nat.1)
    digit_sum(Nat.2, Nat.1) = Nat.1
    digit_sum(Nat.2, Nat.13) = Nat.3
}
