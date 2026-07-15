from nat.nat_base import Nat, div_imp_mod, divides_mul, divides_self, lt_imp_lte_suc, lte_imp_not_lt, trichotomy, add_suc_right
from nat.division import div_mod_decomp, mod_lt, div_of_decomp, mod_of_decomp, div_mul, pos_of_ne_zero
numerals Nat

/// Zero divided by anything is zero.
theorem zero_div(d: Nat) {
    d != Nat.0 implies Nat.0.div(d) = Nat.0
} by {
    if d != Nat.0 {
        pos_of_ne_zero(d)
        div_of_decomp(Nat.0, Nat.0, d)
        Nat.0.div(d) = Nat.0
    }
}

/// When `d` divides `n+1`, the quotient steps up by one.
theorem succ_div_yes(n: Nat, d: Nat) {
    d != Nat.0 and d.divides(n.suc) implies n.suc.div(d) = n.div(d) + Nat.1
} by {
    if d != Nat.0 and d.divides(n.suc) {
        div_mod_decomp(n, d)
        add_suc_right(n.div(d) * d, n.mod(d))
        mod_lt(n, d)
        lt_imp_lte_suc(n.mod(d), d)
        div_imp_mod(n.suc, d)
        if n.mod(d).suc < d {
            mod_of_decomp(n.div(d), n.mod(d).suc, d)
            n.suc.mod(d) = n.mod(d).suc
            false
        }
        lte_imp_not_lt(n.mod(d).suc, d)
        trichotomy(n.mod(d).suc, d)
        div_mul(n.div(d) + Nat.1, d)
        n.suc.div(d) = n.div(d) + Nat.1
    }
}

/// When `d` does not divide `n+1`, the quotient is unchanged.
theorem succ_div_no(n: Nat, d: Nat) {
    d != Nat.0 and not d.divides(n.suc) implies n.suc.div(d) = n.div(d)
} by {
    if d != Nat.0 and not d.divides(n.suc) {
        div_mod_decomp(n, d)
        add_suc_right(n.div(d) * d, n.mod(d))
        mod_lt(n, d)
        lt_imp_lte_suc(n.mod(d), d)
        if n.mod(d).suc = d {
            n.suc = (n.div(d) + Nat.1) * d
            divides_self(d)
            divides_mul(d, n.div(d) + Nat.1, d)
            false
        }
        lte_imp_not_lt(n.mod(d).suc, d)
        trichotomy(n.mod(d).suc, d)
        div_of_decomp(n.div(d), n.mod(d).suc, d)
        n.suc.div(d) = n.div(d)
    }
}

/// The number of multiples of `d` among `1, ..., n`.
define count_multiples(d: Nat, n: Nat) -> Nat {
    match n {
        Nat.zero {
            Nat.0
        }
        Nat.suc(k) {
            if d.divides(n) {
                count_multiples(d, k) + Nat.1
            } else {
                count_multiples(d, k)
            }
        }
    }
}

/// The number of multiples of a nonzero `d` in `1, ..., n` is the quotient `n div d`.
theorem count_multiples_eq_div(d: Nat, n: Nat) {
    d != Nat.0 implies count_multiples(d, n) = n.div(d)
} by {
    if d != Nat.0 {
        let f: Nat -> Bool = function(x: Nat) {
            count_multiples(d, x) = x.div(d)
        }
        zero_div(d)
        f(Nat.0)
        forall(x: Nat) {
            if f(x) {
                count_multiples(d, x) = x.div(d)
                if d.divides(x.suc) {
                    succ_div_yes(x, d)
                    count_multiples(d, x.suc) = count_multiples(d, x) + Nat.1
                    count_multiples(d, x.suc) = x.suc.div(d)
                    f(x.suc)
                }
                if not d.divides(x.suc) {
                    succ_div_no(x, d)
                    count_multiples(d, x.suc) = count_multiples(d, x)
                    f(x.suc)
                }
                f(x.suc)
            }
        }
        f(n)
        count_multiples(d, n) = n.div(d)
    }
}
