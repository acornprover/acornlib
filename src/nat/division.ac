from nat.nat_base import Nat, add_mod, add_cancels_left, lt_diff, lte_mul, trichotomy, lte_imp_not_lt, not_lt_zero
numerals Nat

/// The quotient when dividing `a` by `m` (floor division), characterised by the
/// division decomposition `nat_div(a, m) * m + a.mod(m) = a`. Since `a.mod(0) = a`,
/// the value of `nat_div(a, 0)` is left unconstrained by this equation.
let nat_div(a: Nat, m: Nat) -> q: Nat satisfy {
    q * m + a.mod(m) = a
} by {
    add_mod(a, m)
}

attributes Nat {
    /// The quotient when dividing this number by `m` (floor division).
    let div = nat_div
}

/// The division decomposition: `a = (a div m) * m + (a mod m)`.
theorem div_mod_decomp(a: Nat, m: Nat) {
    a.div(m) * m + a.mod(m) = a
}

/// For a nonzero modulus the remainder is strictly smaller than the modulus.
theorem mod_lt(a: Nat, m: Nat) {
    m != Nat.0 implies a.mod(m) < m
}

/// A nonzero natural is positive.
theorem pos_of_ne_zero(m: Nat) {
    m != Nat.0 implies Nat.0 < m
} by {
    if m != Nat.0 {
        not_lt_zero(m)
        trichotomy(Nat.0, m)
    }
}

/// Uniqueness of the quotient: when `r < m`, the number `q*m + r` has quotient `q`.
theorem div_of_decomp(q: Nat, r: Nat, m: Nat) {
    r < m implies (q * m + r).div(m) = q
} by {
    if r < m {
        m != Nat.0
        let dd: Nat = (q * m + r).div(m)
        div_mod_decomp(q * m + r, m)
        dd * m + (q * m + r).mod(m) = q * m + r
        mod_lt(q * m + r, m)
        if dd < q {
            lt_diff(dd, q)
            let c: Nat satisfy { dd + c = q and c != Nat.0 }
            q * m = (dd + c) * m
            q * m = dd * m + c * m
            dd * m + (q * m + r).mod(m) = dd * m + (c * m + r)
            add_cancels_left(dd * m, (q * m + r).mod(m), c * m + r)
            (q * m + r).mod(m) = c * m + r
            lte_mul(m, c)
            m <= c * m
            c * m <= c * m + r
            m <= (q * m + r).mod(m)
            lte_imp_not_lt(m, (q * m + r).mod(m))
            false
        }
        not (dd < q)
        if q < dd {
            lt_diff(q, dd)
            let c: Nat satisfy { q + c = dd and c != Nat.0 }
            dd * m = (q + c) * m
            dd * m = q * m + c * m
            q * m + (c * m + (q * m + r).mod(m)) = q * m + r
            add_cancels_left(q * m, c * m + (q * m + r).mod(m), r)
            c * m + (q * m + r).mod(m) = r
            lte_mul(m, c)
            m <= c * m
            c * m <= r
            m <= r
            lte_imp_not_lt(m, r)
            false
        }
        trichotomy(dd, q)
        (q * m + r).div(m) = q
    }
}

/// Uniqueness of the remainder: when `r < m`, the number `q*m + r` has remainder `r`.
theorem mod_of_decomp(q: Nat, r: Nat, m: Nat) {
    r < m implies (q * m + r).mod(m) = r
} by {
    if r < m {
        div_of_decomp(q, r, m)
        div_mod_decomp(q * m + r, m)
        add_cancels_left(q * m, (q * m + r).mod(m), r)
        (q * m + r).mod(m) = r
    }
}

/// Multiplication followed by division by the same nonzero number is the identity.
theorem div_mul(q: Nat, m: Nat) {
    m != Nat.0 implies (q * m).div(m) = q
} by {
    if m != Nat.0 {
        pos_of_ne_zero(m)
        div_of_decomp(q, Nat.0, m)
        (q * m).div(m) = q
    }
}
