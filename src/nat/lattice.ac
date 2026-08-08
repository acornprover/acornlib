// Typeclass relations for natural numbers.

from nat.nat_monoid import Nat

// Demonstrating that Nat is a semiring.

from algebra.add_semigroup import AddSemigroup

instance Nat: AddSemigroup

from algebra.add_comm_semigroup import AddCommSemigroup

instance Nat: AddCommSemigroup

from algebra.add_monoid import AddMonoid

instance Nat: AddMonoid

from algebra.add_comm_monoid import AddCommMonoid

instance Nat: AddCommMonoid

from semiring import Semiring

instance Nat: Semiring

from algebra.comm_semigroup import CommSemigroup

instance Nat: CommSemigroup

from algebra.comm_monoid import CommMonoid

instance Nat: CommMonoid

from lattice import Meet, Join, MeetSemilattice, JoinSemilattice, Lattice, DistribLattice
from order import min_lte_left, min_lte_right, lte_min_of_bounds, lte_max_left, lte_max_right,
    max_lte_of_upper_bounds, min_max_distrib_left, max_min_distrib_left

instance Nat: Meet {
    define meet(self, other: Nat) -> Nat {
        self.min(other)
    }
}

instance Nat: Join {
    define join(self, other: Nat) -> Nat {
        self.max(other)
    }
}

theorem nat_meet_lte_left(a: Nat, b: Nat) {
    a.meet(b) <= a
} by {
    min_lte_left(a, b)
}

theorem nat_meet_lte_right(a: Nat, b: Nat) {
    a.meet(b) <= b
} by {
    min_lte_right(a, b)
}

theorem nat_lte_meet_of_bounds(c: Nat, a: Nat, b: Nat) {
    c <= a and c <= b implies c <= a.meet(b)
} by {
    if c <= a and c <= b {
        lte_min_of_bounds(c, a, b)
        c <= a.meet(b)
    }
}

instance Nat: MeetSemilattice

theorem nat_lte_join_left(a: Nat, b: Nat) {
    a <= a.join(b)
} by {
    lte_max_left(a, b)
}

theorem nat_lte_join_right(a: Nat, b: Nat) {
    b <= a.join(b)
} by {
    lte_max_right(a, b)
}

theorem nat_join_lte_of_bounds(a: Nat, b: Nat, c: Nat) {
    a <= c and b <= c implies a.join(b) <= c
} by {
    if a <= c and b <= c {
        max_lte_of_upper_bounds(a, b, c)
        a.join(b) <= c
    }
}

instance Nat: JoinSemilattice

instance Nat: Lattice

theorem nat_meet_join_distrib_left(a: Nat, b: Nat, c: Nat) {
    a.meet(b.join(c)) = a.meet(b).join(a.meet(c))
} by {
    min_max_distrib_left(a, b, c)
}

theorem nat_join_meet_distrib_left(a: Nat, b: Nat, c: Nat) {
    a.join(b.meet(c)) = a.join(b).meet(a.join(c))
} by {
    max_min_distrib_left(a, b, c)
}

instance Nat: DistribLattice
