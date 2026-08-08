from nat.nat_base import Nat
from algebra.comm_monoid import CommMonoid
from algebra.monoid.monoid import Monoid, MonoidHom
from algebra.semigroup import Semigroup
numerals Nat

attributes M: Monoid {
    /// Raises a monoid element to a natural number power using repeated multiplication.
    define pow(self, exp: Nat) -> M {
        match exp {
            Nat.zero {
                M.1
            }
            Nat.suc(n) {
                self * self.pow(n)
            }
        }
    }
}

// Proof that a^1 = a
theorem pow_one[M: Monoid](a: M) {
    a.pow(Nat.1) = a
} by {
     // by definition of Nat.1
     // by definition of pow
     // by definition of pow (base case)
     // by mul_identity_right
     // transitivity
}

theorem pow_zero[M: Monoid](a: M) {
    a.pow(Nat.0) = M.1
}

theorem alt_pow_zero[M: Monoid](a: M, exp: Nat) {
    exp != Nat.0 or a.pow(exp) = M.1
}

// Proof that a^n * a^m = a^(n+m)
theorem pow_add[M: Monoid](a: M, n: Nat, m: Nat) {
    a.pow(n) * a.pow(m) = a.pow(n + m)
} by {
    // Define a helper function for induction
    define f(x: Nat) -> Bool { a.pow(x) * a.pow(m) = a.pow(x + m) }

    // Base case: a^0 * a^m = a^(0+m)
    a.pow(Nat.0) = M.1
    M.1 * a.pow(m) = a.pow(m)
    Nat.0 + m = m
    a.pow(Nat.0) * a.pow(m) = a.pow(Nat.0 + m)
    f(Nat.0)

    // Inductive step
    forall(x: Nat) {
        if f(x) {
            // Induction hypothesis
            a.pow(x) * a.pow(m) = a.pow(x + m)
            a * a.pow(x) * a.pow(m) = a * (a.pow(x) * a.pow(m))
            // Now prove for x.suc
            a.pow(x.suc) * a.pow(m) = a.pow((x + m).suc)
            f(x.suc)
        }
    }

}

// Proof that (a^n)^m = a ^ (n * m)
theorem pow_pow[M: Monoid](a: M, n: Nat, m: Nat) {
    a.pow(n).pow(m) = a.pow(n * m)
} by {
    // Define a helper function for induction
    define f(x: Nat) -> Bool { a.pow(n).pow(x) = a.pow(n * x) }

    // Base case
    n * Nat.0 = Nat.0
    a.pow(n).pow(Nat.0) = M.1
    a.pow(n * Nat.0) = M.1
    a.pow(n).pow(Nat.0) = a.pow(n * Nat.0)
    f(Nat.0)

    // Inductive step
    forall(x: Nat) {
        if f(x) {
            a.pow(n).pow(x) = a.pow(n * x)
            a.pow(n).pow(x.suc) = a.pow(n * x + n)
            f(x.suc)
        }
    }
}

// Proof that 1^n = 1
theorem one_pow[M: Monoid](n: Nat) {
    M.1.pow(n) = M.1
} by {
    define f(x: Nat) -> Bool { one_pow[M](x) }

    M.1.pow(Nat.0) = M.1
    f(Nat.0)

    forall(x: Nat) {
        if f(x) {
            M.1.pow(x) = M.1
            M.1.pow(x.suc) = M.1
            f(x.suc)
        }
    }
}

/// A monoid homomorphism preserves powers.
theorem monoid_hom_pow[M: Monoid, N: Monoid](f: MonoidHom[M, N], a: M, n: Nat) {
    f.hom(a.pow(n)) = f.hom(a).pow(n)
} by {
    define p(k: Nat) -> Bool {
        f.hom(a.pow(k)) = f.hom(a).pow(k)
    }

    a.pow(Nat.0) = M.1
    f.hom(M.1) = N.1
    f.hom(a).pow(Nat.0) = N.1
    p(Nat.0)

    forall(k: Nat) {
        if p(k) {
            f.hom(a * a.pow(k)) = f.hom(a) * f.hom(a.pow(k))
            f.hom(a) * f.hom(a.pow(k)) = f.hom(a) * f.hom(a).pow(k)
            p(k.suc)
        }
    }
}

theorem pow_distrib_mul[M: CommMonoid](a: M, b: M, c: Nat) {
    (a * b).pow(c) = a.pow(c) * b.pow(c)
} by {
    // Inductive proof on c
    define f(x: Nat) -> Bool {
        (a * b).pow(x) = a.pow(x) * b.pow(x)
    }

    // Base case: c = Nat.0
    // (a * b).pow(Nat.0) is M.1 by definition of pow.
    // a.pow(Nat.0) is M.1 by definition of pow.
    // b.pow(Nat.0) is M.1 by definition of pow.
    // So, f(Nat.0) becomes M.1 = M.1 * M.1.
    // This is true by mul_identity_right(M.1) or mul_identity_left(M.1) from Monoid.
    f(Nat.0)

    // Inductive step: Assume f(k), prove f(k.suc)
    forall(k: Nat) {
        if f(k) {
            // Inductive Hypothesis (IH):
            // (a * b).pow(k) = a.pow(k) * b.pow(k)

            // Goal: (a * b).pow(k.suc) = a.pow(k.suc) * b.pow(k.suc)

            // LHS expansion: (a * b).pow(k.suc) = (a * b) * (a * b).pow(k)
            // By IH: (a * b).pow(k.suc) = (a * b) * (a.pow(k) * b.pow(k))

            // RHS expansion: a.pow(k.suc) * b.pow(k.suc) = (a * a.pow(k)) * (b * b.pow(k))

            // Proof: Show expanded LHS equals expanded RHS using Commutativity and Associativity.
            // (a * b) * (a.pow(k) * b.pow(k)) = (a * a.pow(k)) * (b * b.pow(k))
            // This relies on the Acorn prover's ability to apply these properties for CommMonoid.
            (a * b).pow(k.suc) = (a * b) * (a * b).pow(k)
            (a * b).pow(k) = a.pow(k) * b.pow(k)
            (a * b).pow(k.suc) = (a * b) * (a.pow(k) * b.pow(k))
            (a * b) * (a.pow(k) * b.pow(k)) = (a * a.pow(k)) * (b * b.pow(k))

            // Therefore, using definitions of pow and the equality above:
            f(k.suc)
        }
    }
     // Apply induction
}

instance Nat: Semigroup

instance Nat: Monoid

// attributes Nat {
//     let exp = Nat.pow
// }

theorem exp_one(a: Nat) {
    a.pow(1) = a
}

theorem exp_zero(a: Nat) {
    a.pow(0) = 1
}

theorem exp_add(a: Nat, b: Nat, c: Nat) {
    a.pow(b + c) = a.pow(b) * a.pow(c)
} by {
    // Inductive step
    let f: Nat -> Bool = function(x: Nat) {
        a.pow(b + x) = a.pow(b) * a.pow(x)
    }
}

theorem exp_mul(a: Nat, b: Nat, c: Nat) {
    a.pow(b * c) = a.pow(b).pow(c)
} by {
    // Inductive step
    let f: Nat -> Bool = function(x: Nat) {
        a.pow(b * x) = a.pow(b).pow(x)
    }
}

theorem zero_exp(n: Nat) {
    n != 0 implies 0.pow(n) = 0
} by {
    let k: Nat satisfy {
        n = k.suc
    }
    Nat.0.pow(n) = Nat.0 * Nat.0.pow(k)
}

theorem one_exp(n: Nat) {
    1.pow(n) = 1
}

theorem exp_gte_one(a: Nat, b: Nat) {
    a != 0 implies 1 <= a.pow(b)
} by {
    // Induction step
    let f: Nat -> Bool = function(x: Nat) {
        1 <= a.pow(x)
    }
    f(0)
    forall(x: Nat) {
        if f(x) {
            a.pow(x) <= a.pow(x.suc)
            f(x.suc)
        }
    }
}

theorem exp_gt_one(a: Nat, b: Nat) {
    1 < a and b != 0 implies 1 < a.pow(b)
} by {
    let b_pred: Nat satisfy {
        b = b_pred.suc
    }
    a.pow(b) = a * a.pow(b_pred)
    1 <= a.pow(b_pred)
}

theorem exp_ne_zero(a: Nat, b: Nat) {
    a != 0 implies a.pow(b) != 0
} by {
    1 <= a.pow(b)
}

theorem lte_imp_exp_lte(a: Nat, b: Nat, c: Nat) {
    a != 0 and b <= c implies a.pow(b) <= a.pow(c)
} by {
    let d: Nat satisfy {
        b + d = c
    }
    a.pow(d) != Nat.0
    a.pow(b) * a.pow(d) = a.pow(b + d)
    a.pow(b) <= a.pow(b) * a.pow(d)
}

theorem lte_exp(a: Nat, b: Nat) {
    a != 0 and b != 0 implies a <= a.pow(b)
}

theorem lt_imp_exp_lt(a: Nat, b: Nat, c: Nat) {
    1 < a and b < c implies a.pow(b) < a.pow(c)
} by {
    let d: Nat satisfy {
        b + d = c
    }
    1 < a.pow(d)
    a.pow(b) * a.pow(d) = a.pow(b + d)
    a.pow(b) != Nat.0
}

theorem exp_lte_imp_lte(a: Nat, b: Nat, c: Nat) {
    1 < a and a.pow(b) <= a.pow(c)
    implies
    b <= c
}

theorem exp_lt_imp_lt(a: Nat, b: Nat, c: Nat) {
    1 < a and a.pow(b) < a.pow(c)
    implies
    b < c
}

theorem exp_eq_one_imp(a: Nat, b: Nat) {
    a != 1 and a.pow(b) = 1 implies b = 0
} by {
    b = 0 or exists(c: Nat) { b = c.suc }
    if exists(c: Nat) { b = c.suc } {
        let c: Nat satisfy { b = c.suc }
        a.pow(c.suc) = a.pow(c) * a
        a.pow(c) * a = 1
        a = 1
    }
}

theorem sq_eq_mul(a: Nat) {
    a.pow(2) = a * a
}
