from nat.nat_base import Nat, add_assoc, add_cancels_left, add_cancels_right,
    add_comm, add_comm_4, add_to_zero, add_zero_left, add_zero_right,
    distrib_left, distrib_right, lte_cancel_suc, lte_ref, lte_trans, lt_diff,
    lt_imp_lte_suc, mul_cancel_left, mul_comm, mul_one_left, mul_zero_left,
    only_zero_lte_zero, strong_induction, suc_sub_one, true_below,
    true_below_apply, zero_or_suc
from nat.digit_sum import digit_sum, digit_sum_zero, div_lt
from nat.carry import carry_add_div, carry_count_fuel, carry_count_fuel_comm,
    carry_count_fuel_suc, carry_count_fuel_zero, carry_count_fuel_zero_left,
    carry_count_fuel_zero_right, carry_digit, carry_high, carry_high_fuel,
    carry_high_fuel_comm, carry_high_fuel_suc, carry_high_fuel_zero,
    carry_out, carry_out_comm, carry_out_zero_left, carry_out_zero_right,
    carry_total, carry_total_decomp, carry_total_eq, digit_sum_carry_step,
    digit_sum_mod_div, is_addition_carry_count, iterated_div,
    iterated_div_suc, iterated_div_zero, iterated_div_zero_value
numerals Nat

/// If a three-term natural sum is zero, then each term is zero.
theorem add_three_eq_zero(a: Nat, b: Nat, c: Nat) {
    a + b + c = Nat.0 implies a = Nat.0 and b = Nat.0 and c = Nat.0
} by {
    if a + b + c = Nat.0 {
        add_to_zero(a + b, c)
        add_to_zero(a, b)
        a = Nat.0 and b = Nat.0 and c = Nat.0
    }
}

/// With zero fuel, vanishing high state forces every summand to vanish.
theorem carry_high_fuel_zero_imp_inputs_zero(p: Nat, a: Nat, b: Nat, c: Nat) {
    carry_high_fuel(p, a, b, c, Nat.0) = Nat.0 implies
        a = Nat.0 and b = Nat.0 and c = Nat.0
} by {
    if carry_high_fuel(p, a, b, c, Nat.0) = Nat.0 {
        carry_high_fuel_zero(p, a, b, c)
        a + b + c = Nat.0
        add_three_eq_zero(a, b, c)
        a = Nat.0 and b = Nat.0 and c = Nat.0
    }
}

/// With zero fuel and vanishing high state, the carry-count balance is
/// degenerate.
theorem carry_count_fuel_balance_zero(p: Nat, a: Nat, b: Nat, c: Nat) {
    carry_high_fuel(p, a, b, c, Nat.0) = Nat.0 implies
        p * carry_count_fuel(p, a, b, c, Nat.0) + digit_sum(p, a + b + c) =
            carry_count_fuel(p, a, b, c, Nat.0) +
            digit_sum(p, a) + digit_sum(p, b) + c
} by {
    if carry_high_fuel(p, a, b, c, Nat.0) = Nat.0 {
        carry_high_fuel_zero_imp_inputs_zero(p, a, b, c)
        carry_count_fuel_zero(p, a, b, c)
        digit_sum_zero(p)
        p * carry_count_fuel(p, a, b, c, Nat.0) = Nat.0
        p * carry_count_fuel(p, a, b, c, Nat.0) + digit_sum(p, a + b + c) =
            carry_count_fuel(p, a, b, c, Nat.0) +
            digit_sum(p, a) + digit_sum(p, b) + c
    }
}

/// The first two terms of a right-associated sum may be swapped.
theorem add_swap_left(a: Nat, b: Nat, c: Nat) {
    a + (b + c) = b + (a + c)
} by {
    add_comm_4(Nat.0, a, b, c)
    add_zero_left(a)
    add_zero_left(b)
}

/// True if a recursion budget forces repeated division by `p` to reach zero.
define iterated_div_large_case(p: Nat, n: Nat, fuel: Nat) -> Bool {
    Nat.1 < p and n <= fuel implies iterated_div(p, n, fuel) = Nat.0
}

/// In a base greater than one, at most `n` repeated divisions of `n` reach
/// zero.
theorem iterated_div_large(p: Nat, n: Nat, fuel: Nat) {
    Nat.1 < p and n <= fuel implies iterated_div(p, n, fuel) = Nat.0
} by {
    let f: Nat -> Bool = function(m: Nat) {
        forall(fuel2: Nat) {
            iterated_div_large_case(p, m, fuel2)
        }
    }
    strong_induction(f)
    forall(m: Nat) {
        if true_below(f, m) {
            forall(fuel2: Nat) {
                if Nat.1 < p and m <= fuel2 {
                    p != Nat.0
                    if m = Nat.0 {
                        iterated_div_zero_value(p, fuel2)
                        iterated_div(p, m, fuel2) = Nat.0
                    }
                    if m != Nat.0 {
                        zero_or_suc(m)
                        let k: Nat satisfy { m = k.suc }
                        zero_or_suc(fuel2)
                        if fuel2 = Nat.0 {
                            only_zero_lte_zero(m)
                            false
                        }
                        let h: Nat satisfy { fuel2 = h.suc }
                        let q: Nat = m.div(p)
                        div_lt(m, p)
                        q < m
                        lt_imp_lte_suc(q, m)
                        lte_cancel_suc(q, k)
                        lte_cancel_suc(k, h)
                        k <= h
                        lte_trans(q, k, h)
                        true_below_apply(f, m, q)
                        iterated_div_large_case(p, q, h)
                        Nat.1 < p and q <= h
                        iterated_div_suc(p, m, h)
                        iterated_div(p, m, fuel2) = iterated_div(p, q, h)
                        iterated_div(p, m, fuel2) = Nat.0
                    }
                    iterated_div_large_case(p, m, fuel2)
                }
            }
            forall(fuel2: Nat) {
                iterated_div_large_case(p, m, fuel2)
            }
            f(m)
        }
    }
    f(n)
    iterated_div_large_case(p, n, fuel)
}

/// Repeated division of `n` by a base greater than one reaches zero after
/// exactly the budget `n`.
theorem iterated_div_self_zero(p: Nat, n: Nat) {
    Nat.1 < p implies iterated_div(p, n, n) = Nat.0
} by {
    if Nat.1 < p {
        lte_ref(n)
        iterated_div_large(p, n, n)
        iterated_div(p, n, n) = Nat.0
    }
}

/// The incoming carry remaining after a finite number of carry steps.
define carry_in_fuel(p: Nat, a: Nat, b: Nat, c: Nat, fuel: Nat) -> Nat {
    match fuel {
        Nat.zero {
            c
        }
        Nat.suc(k) {
            carry_in_fuel(p, a.div(p), b.div(p), carry_out(p, a, b, c), k)
        }
    }
}

/// The zero fuel incoming carry is the initial carry.
theorem carry_in_fuel_zero(p: Nat, a: Nat, b: Nat, c: Nat) {
    carry_in_fuel(p, a, b, c, Nat.0) = c
}

/// One step of the remaining incoming-carry recursion.
theorem carry_in_fuel_suc(p: Nat, a: Nat, b: Nat, c: Nat, fuel: Nat) {
    carry_in_fuel(p, a, b, c, fuel.suc) =
        carry_in_fuel(p, a.div(p), b.div(p), carry_out(p, a, b, c), fuel)
}

/// The remaining incoming carry is symmetric in the two addends.
theorem carry_in_fuel_comm(p: Nat, a: Nat, b: Nat, c: Nat, fuel: Nat) {
    carry_in_fuel(p, a, b, c, fuel) = carry_in_fuel(p, b, a, c, fuel)
} by {
    define pred(k: Nat) -> Bool {
        forall(x: Nat, y: Nat, z: Nat) {
            carry_in_fuel(p, x, y, z, k) = carry_in_fuel(p, y, x, z, k)
        }
    }

    forall(x: Nat, y: Nat, z: Nat) {
        carry_in_fuel(p, x, y, z, Nat.0) = carry_in_fuel(p, y, x, z, Nat.0)
    }
    pred(Nat.0)

    forall(k: Nat) {
        if pred(k) {
            forall(x: Nat, y: Nat, z: Nat) {
                carry_in_fuel_suc(p, x, y, z, k)
                carry_in_fuel_suc(p, y, x, z, k)
                carry_out_comm(p, x, y, z)
                let h: Bool =
                    carry_in_fuel(p, x.div(p), y.div(p),
                        carry_out(p, x, y, z), k) =
                    carry_in_fuel(p, y.div(p), x.div(p),
                        carry_out(p, x, y, z), k)
                h
                carry_in_fuel(p, x, y, z, k.suc) =
                    carry_in_fuel(p, y, x, z, k.suc)
            }
            forall(x: Nat, y: Nat, z: Nat) {
                carry_in_fuel(p, x, y, z, k.suc) =
                    carry_in_fuel(p, y, x, z, k.suc)
            }
            pred(k.suc)
        }
    }
    pred(fuel)
    let h: Bool =
        carry_in_fuel(p, a, b, c, fuel) = carry_in_fuel(p, b, a, c, fuel)
}

/// With a zero right addend and no incoming carry, the remaining incoming
/// carry is zero.
theorem carry_in_fuel_zero_right(p: Nat, a: Nat, fuel: Nat) {
    p != Nat.0 implies carry_in_fuel(p, a, Nat.0, Nat.0, fuel) = Nat.0
} by {
    define pred(k: Nat) -> Bool {
        forall(x: Nat) {
            carry_in_fuel(p, x, Nat.0, Nat.0, k) = Nat.0
        }
    }

    forall(x: Nat) {
        carry_in_fuel(p, x, Nat.0, Nat.0, Nat.0) = Nat.0
    }
    pred(Nat.0)

    forall(k: Nat) {
        if pred(k) {
            forall(x: Nat) {
                if p != Nat.0 {
                    carry_in_fuel_suc(p, x, Nat.0, Nat.0, k)
                    carry_out_zero_right(p, x)
                    iterated_div_zero_value(p, Nat.1)
                    Nat.0.div(p) = Nat.0
                    let h: Bool =
                        carry_in_fuel(p, x.div(p), Nat.0, Nat.0, k) = Nat.0
                    h
                    carry_in_fuel(p, x, Nat.0, Nat.0, k.suc) = Nat.0
                }
            }
            forall(x: Nat) {
                if p != Nat.0 {
                    carry_in_fuel(p, x, Nat.0, Nat.0, k.suc) = Nat.0
                }
            }
            if p != Nat.0 {
                pred(k.suc)
            }
        }
    }

    if p != Nat.0 {
        pred(fuel)
        let h: Bool = carry_in_fuel(p, a, Nat.0, Nat.0, fuel) = Nat.0
        carry_in_fuel(p, a, Nat.0, Nat.0, fuel) = Nat.0
    }
}

/// With a zero left addend and no incoming carry, the remaining incoming carry
/// is zero.
theorem carry_in_fuel_zero_left(p: Nat, b: Nat, fuel: Nat) {
    p != Nat.0 implies carry_in_fuel(p, Nat.0, b, Nat.0, fuel) = Nat.0
} by {
    if p != Nat.0 {
        carry_in_fuel_comm(p, Nat.0, b, Nat.0, fuel)
        carry_in_fuel_zero_right(p, b, fuel)
        carry_in_fuel(p, Nat.0, b, Nat.0, fuel) = Nat.0
    }
}

/// The finite high state decomposes into the residual quotients of the two
/// addends and the residual incoming carry.
theorem carry_high_fuel_eq_residual_sum(
    p: Nat, a: Nat, b: Nat, c: Nat, fuel: Nat
) {
    carry_high_fuel(p, a, b, c, fuel) =
        iterated_div(p, a, fuel) + iterated_div(p, b, fuel) +
        carry_in_fuel(p, a, b, c, fuel)
} by {
    define pred(k: Nat) -> Bool {
        forall(x: Nat, y: Nat, z: Nat) {
            carry_high_fuel(p, x, y, z, k) =
                iterated_div(p, x, k) + iterated_div(p, y, k) +
                carry_in_fuel(p, x, y, z, k)
        }
    }

    forall(x: Nat, y: Nat, z: Nat) {
        carry_high_fuel_zero(p, x, y, z)
        iterated_div_zero(p, x)
        iterated_div_zero(p, y)
        carry_in_fuel_zero(p, x, y, z)
        carry_high_fuel(p, x, y, z, Nat.0) =
            iterated_div(p, x, Nat.0) + iterated_div(p, y, Nat.0) +
            carry_in_fuel(p, x, y, z, Nat.0)
    }
    pred(Nat.0)

    forall(k: Nat) {
        if pred(k) {
            forall(x: Nat, y: Nat, z: Nat) {
                carry_high_fuel_suc(p, x, y, z, k)
                let h: Bool =
                    carry_high_fuel(p, x.div(p), y.div(p),
                        carry_out(p, x, y, z), k) =
                        iterated_div(p, x.div(p), k) +
                        iterated_div(p, y.div(p), k) +
                        carry_in_fuel(p, x.div(p), y.div(p),
                            carry_out(p, x, y, z), k)
                h
                iterated_div_suc(p, x, k)
                iterated_div_suc(p, y, k)
                carry_in_fuel_suc(p, x, y, z, k)
                carry_high_fuel(p, x, y, z, k.suc) =
                    iterated_div(p, x, k.suc) + iterated_div(p, y, k.suc) +
                    carry_in_fuel(p, x, y, z, k.suc)
            }
            forall(x: Nat, y: Nat, z: Nat) {
                carry_high_fuel(p, x, y, z, k.suc) =
                    iterated_div(p, x, k.suc) + iterated_div(p, y, k.suc) +
                    carry_in_fuel(p, x, y, z, k.suc)
            }
            pred(k.suc)
        }
    }
    pred(fuel)
    let h: Bool =
        carry_high_fuel(p, a, b, c, fuel) =
            iterated_div(p, a, fuel) + iterated_div(p, b, fuel) +
            carry_in_fuel(p, a, b, c, fuel)
}

/// If the high state has vanished, then both residual quotients and the
/// residual incoming carry have vanished.
theorem carry_high_fuel_zero_imp_residuals_zero(
    p: Nat, a: Nat, b: Nat, c: Nat, fuel: Nat
) {
    carry_high_fuel(p, a, b, c, fuel) = Nat.0 implies
        iterated_div(p, a, fuel) = Nat.0 and
        iterated_div(p, b, fuel) = Nat.0 and
        carry_in_fuel(p, a, b, c, fuel) = Nat.0
} by {
    if carry_high_fuel(p, a, b, c, fuel) = Nat.0 {
        carry_high_fuel_eq_residual_sum(p, a, b, c, fuel)
        add_three_eq_zero(iterated_div(p, a, fuel), iterated_div(p, b, fuel),
            carry_in_fuel(p, a, b, c, fuel))
        iterated_div(p, b, fuel) = Nat.0
        iterated_div(p, a, fuel) = Nat.0 and
            iterated_div(p, b, fuel) = Nat.0
        iterated_div(p, a, fuel) = Nat.0 and
            iterated_div(p, b, fuel) = Nat.0 and
            carry_in_fuel(p, a, b, c, fuel) = Nat.0
    }
}

/// If the high state has vanished, then the left residual quotient has
/// vanished.
theorem carry_high_fuel_zero_imp_left_residual_zero(
    p: Nat, a: Nat, b: Nat, c: Nat, fuel: Nat
) {
    carry_high_fuel(p, a, b, c, fuel) = Nat.0 implies
        iterated_div(p, a, fuel) = Nat.0
} by {
    if carry_high_fuel(p, a, b, c, fuel) = Nat.0 {
        carry_high_fuel_zero_imp_residuals_zero(p, a, b, c, fuel)
        iterated_div(p, a, fuel) = Nat.0
    }
}

/// If the high state has vanished, then the right residual quotient has
/// vanished.
theorem carry_high_fuel_zero_imp_right_residual_zero(
    p: Nat, a: Nat, b: Nat, c: Nat, fuel: Nat
) {
    carry_high_fuel(p, a, b, c, fuel) = Nat.0 implies
        iterated_div(p, b, fuel) = Nat.0
} by {
    if carry_high_fuel(p, a, b, c, fuel) = Nat.0 {
        carry_high_fuel_zero_imp_residuals_zero(p, a, b, c, fuel)
        iterated_div(p, b, fuel) = Nat.0
    }
}

/// If the high state has vanished, then the residual incoming carry has
/// vanished.
theorem carry_high_fuel_zero_imp_carry_residual_zero(
    p: Nat, a: Nat, b: Nat, c: Nat, fuel: Nat
) {
    carry_high_fuel(p, a, b, c, fuel) = Nat.0 implies
        carry_in_fuel(p, a, b, c, fuel) = Nat.0
} by {
    if carry_high_fuel(p, a, b, c, fuel) = Nat.0 {
        carry_high_fuel_zero_imp_residuals_zero(p, a, b, c, fuel)
        carry_in_fuel(p, a, b, c, fuel) = Nat.0
    }
}

/// Rearrangement used in the one-step carry-count balance.
theorem carry_count_balance_rearrange(
    am: Nat, bm: Nat, c: Nat, tail: Nat, da: Nat, db: Nat, co: Nat
) {
    (am + bm + c) + (tail + da + db + co) =
        (co + tail) + (am + da) + (bm + db) + c
} by {
    add_assoc(am + bm, c, tail + da + db + co)
    add_assoc(am, bm, c + (tail + da + db + co))
    add_assoc(tail + da, db, co)
    add_assoc(tail, da, db + co)

    add_comm(db, co)
    add_swap_left(da, co, db)
    add_swap_left(tail, co, da + db)
    add_swap_left(c, co, tail + (da + db))
    add_swap_left(bm, co, c + (tail + (da + db)))
    add_swap_left(am, co, bm + (c + (tail + (da + db))))

    c + (tail + (da + db)) = c + (tail + (da + db))
    add_swap_left(c, tail, da + db)
    add_swap_left(bm, tail, c + (da + db))
    add_swap_left(am, tail, bm + (c + (da + db)))

    add_swap_left(c, da, db)
    add_swap_left(bm, da, c + db)
    add_comm(c, db)

    add_assoc(co, tail, am + (da + (bm + (db + c))))
    add_assoc(am, da, bm + (db + c))
    add_assoc(bm, db, c)
    (co + tail) + (am + da) + (bm + db) + c =
        co + tail + (am + da) + (bm + db) + c
    co + tail + (am + da + (bm + db + c)) =
        co + tail + (am + (da + (bm + (db + c))))
}

/// One carry-count step lifts a balanced tail equation to the previous digit.
theorem carry_count_fuel_balance_step(
    p: Nat, a: Nat, b: Nat, c: Nat, tail_count: Nat
) {
    Nat.1 < p and
        p * tail_count +
            digit_sum(p, a.div(p) + b.div(p) + carry_out(p, a, b, c)) =
        tail_count + digit_sum(p, a.div(p)) + digit_sum(p, b.div(p)) +
            carry_out(p, a, b, c) implies
        p * (carry_out(p, a, b, c) + tail_count) + digit_sum(p, a + b + c) =
            (carry_out(p, a, b, c) + tail_count) +
            digit_sum(p, a) + digit_sum(p, b) + c
} by {
    if Nat.1 < p and
        p * tail_count +
            digit_sum(p, a.div(p) + b.div(p) + carry_out(p, a, b, c)) =
        tail_count + digit_sum(p, a.div(p)) + digit_sum(p, b.div(p)) +
            carry_out(p, a, b, c) {
        let co: Nat = carry_out(p, a, b, c)
        let digit: Nat = carry_digit(p, a, b, c)
        let aq: Nat = a.div(p)
        let bq: Nat = b.div(p)
        let tail_sum: Nat = digit_sum(p, aq + bq + co)
        let da: Nat = digit_sum(p, aq)
        let db: Nat = digit_sum(p, bq)

        digit_sum_carry_step(p, a, b, c)
        digit_sum(p, a + b + c) =
            digit + digit_sum(p, carry_high(p, a, b, c))
        carry_high(p, a, b, c) = aq + bq + co
        digit_sum(p, carry_high(p, a, b, c)) = tail_sum
        digit_sum(p, a + b + c) = digit + tail_sum

        digit_sum_mod_div(p, a)
        digit_sum(p, a) = a.mod(p) + da
        digit_sum_mod_div(p, b)
        digit_sum(p, b) = b.mod(p) + db

        carry_total_decomp(p, a, b, c)
        co * p + digit = carry_total(p, a, b, c)
        carry_total_eq(p, a, b, c)
        carry_total(p, a, b, c) = a.mod(p) + b.mod(p) + c
        co * p + digit = a.mod(p) + b.mod(p) + c
        mul_comm(p, co)
        p * co = co * p
        p * co + digit = a.mod(p) + b.mod(p) + c

        distrib_left(p, co, tail_count)
        p * (co + tail_count) = p * co + p * tail_count

        (p * co + digit) + (p * tail_count + tail_sum) =
            (a.mod(p) + b.mod(p) + c) + (tail_count + da + db + co)
        carry_count_balance_rearrange(a.mod(p), b.mod(p), c, tail_count, da, db, co)
        p * (carry_out(p, a, b, c) + tail_count) + digit_sum(p, a + b + c) =
            (carry_out(p, a, b, c) + tail_count) +
            digit_sum(p, a) + digit_sum(p, b) + c
    }
}

/// If the finite carry process has no remaining high state, then its carry
/// count satisfies the digit-sum balance, including an incoming carry.
theorem carry_count_fuel_balance_of_high_zero(
    p: Nat, a: Nat, b: Nat, c: Nat, fuel: Nat
) {
    Nat.1 < p and carry_high_fuel(p, a, b, c, fuel) = Nat.0 implies
        p * carry_count_fuel(p, a, b, c, fuel) + digit_sum(p, a + b + c) =
            carry_count_fuel(p, a, b, c, fuel) +
            digit_sum(p, a) + digit_sum(p, b) + c
} by {
    define pred(k: Nat) -> Bool {
        forall(x: Nat, y: Nat, z: Nat) {
            Nat.1 < p and carry_high_fuel(p, x, y, z, k) = Nat.0 implies
                p * carry_count_fuel(p, x, y, z, k) +
                    digit_sum(p, x + y + z) =
                carry_count_fuel(p, x, y, z, k) +
                    digit_sum(p, x) + digit_sum(p, y) + z
        }
    }

    forall(x: Nat, y: Nat, z: Nat) {
        if Nat.1 < p and carry_high_fuel(p, x, y, z, Nat.0) = Nat.0 {
            carry_count_fuel_balance_zero(p, x, y, z)
            p * carry_count_fuel(p, x, y, z, Nat.0) +
                digit_sum(p, x + y + z) =
                carry_count_fuel(p, x, y, z, Nat.0) +
                digit_sum(p, x) + digit_sum(p, y) + z
        }
    }
    pred(Nat.0)

    forall(k: Nat) {
        if pred(k) {
            forall(x: Nat, y: Nat, z: Nat) {
                if Nat.1 < p and carry_high_fuel(p, x, y, z, k.suc) = Nat.0 {
                    carry_high_fuel_suc(p, x, y, z, k)
                    let tail_h: Bool =
                        p * carry_count_fuel(p, x.div(p), y.div(p),
                            carry_out(p, x, y, z), k) +
                            digit_sum(p, x.div(p) + y.div(p) +
                                carry_out(p, x, y, z)) =
                        carry_count_fuel(p, x.div(p), y.div(p),
                            carry_out(p, x, y, z), k) +
                            digit_sum(p, x.div(p)) + digit_sum(p, y.div(p)) +
                            carry_out(p, x, y, z)
                    tail_h
                    carry_count_fuel_suc(p, x, y, z, k)
                    carry_count_fuel_balance_step(p, x, y, z,
                        carry_count_fuel(p, x.div(p), y.div(p),
                            carry_out(p, x, y, z), k))
                    p * (carry_out(p, x, y, z) +
                        carry_count_fuel(p, x.div(p), y.div(p),
                            carry_out(p, x, y, z), k)) +
                        digit_sum(p, x + y + z) =
                        (carry_out(p, x, y, z) +
                        carry_count_fuel(p, x.div(p), y.div(p),
                            carry_out(p, x, y, z), k)) +
                        digit_sum(p, x) + digit_sum(p, y) + z
                    p * carry_count_fuel(p, x, y, z, k.suc) +
                        digit_sum(p, x + y + z) =
                        carry_count_fuel(p, x, y, z, k.suc) +
                        digit_sum(p, x) + digit_sum(p, y) + z
                }
            }
            forall(x: Nat, y: Nat, z: Nat) {
                if Nat.1 < p and carry_high_fuel(p, x, y, z, k.suc) = Nat.0 {
                    p * carry_count_fuel(p, x, y, z, k.suc) +
                        digit_sum(p, x + y + z) =
                        carry_count_fuel(p, x, y, z, k.suc) +
                        digit_sum(p, x) + digit_sum(p, y) + z
                }
            }
            pred(k.suc)
        }
    }

    if Nat.1 < p and carry_high_fuel(p, a, b, c, fuel) = Nat.0 {
        pred(fuel)
        let h: Bool =
            p * carry_count_fuel(p, a, b, c, fuel) +
                digit_sum(p, a + b + c) =
            carry_count_fuel(p, a, b, c, fuel) +
                digit_sum(p, a) + digit_sum(p, b) + c
        p * carry_count_fuel(p, a, b, c, fuel) + digit_sum(p, a + b + c) =
            carry_count_fuel(p, a, b, c, fuel) +
            digit_sum(p, a) + digit_sum(p, b) + c
    }
}

/// Once the finite carry process for `a + b` has no remaining high state,
/// its recursive carry count satisfies the carry-count predicate.
theorem carry_count_fuel_is_addition_carry_count(
    p: Nat, a: Nat, b: Nat, fuel: Nat
) {
    Nat.1 < p and carry_high_fuel(p, a, b, Nat.0, fuel) = Nat.0 implies
        is_addition_carry_count(
            p, a, b, carry_count_fuel(p, a, b, Nat.0, fuel))
} by {
    if Nat.1 < p and carry_high_fuel(p, a, b, Nat.0, fuel) = Nat.0 {
        carry_count_fuel_balance_of_high_zero(p, a, b, Nat.0, fuel)
        p * carry_count_fuel(p, a, b, Nat.0, fuel) +
            digit_sum(p, a + b + Nat.0) =
            carry_count_fuel(p, a, b, Nat.0, fuel) +
            digit_sum(p, a) + digit_sum(p, b) + Nat.0
        p * carry_count_fuel(p, a, b, Nat.0, fuel) +
            digit_sum(p, a + b) =
            carry_count_fuel(p, a, b, Nat.0, fuel) +
            digit_sum(p, a) + digit_sum(p, b)
        is_addition_carry_count(
            p, a, b, carry_count_fuel(p, a, b, Nat.0, fuel))
    }
}

/// The finite high carry state is repeated division of the whole sum.
theorem carry_high_fuel_eq_iterated_div_total(
    p: Nat, a: Nat, b: Nat, c: Nat, fuel: Nat
) {
    p != Nat.0 implies
        carry_high_fuel(p, a, b, c, fuel) =
            iterated_div(p, a + b + c, fuel)
} by {
    define pred(k: Nat) -> Bool {
        forall(x: Nat, y: Nat, z: Nat) {
            p != Nat.0 implies
                carry_high_fuel(p, x, y, z, k) =
                    iterated_div(p, x + y + z, k)
        }
    }

    forall(x: Nat, y: Nat, z: Nat) {
        if p != Nat.0 {
            carry_high_fuel_zero(p, x, y, z)
            iterated_div_zero(p, x + y + z)
            carry_high_fuel(p, x, y, z, Nat.0) =
                iterated_div(p, x + y + z, Nat.0)
        }
    }
    pred(Nat.0)

    forall(k: Nat) {
        if pred(k) {
            forall(x: Nat, y: Nat, z: Nat) {
                if p != Nat.0 {
                    carry_high_fuel_suc(p, x, y, z, k)
                    let h: Bool =
                        carry_high_fuel(p, x.div(p), y.div(p),
                            carry_out(p, x, y, z), k) =
                            iterated_div(p, x.div(p) + y.div(p) +
                                carry_out(p, x, y, z), k)
                    h
                    carry_add_div(p, x, y, z)
                    iterated_div_suc(p, x + y + z, k)
                    carry_high_fuel(p, x, y, z, k.suc) =
                        iterated_div(p, x + y + z, k.suc)
                }
            }
            forall(x: Nat, y: Nat, z: Nat) {
                if p != Nat.0 {
                    carry_high_fuel(p, x, y, z, k.suc) =
                        iterated_div(p, x + y + z, k.suc)
                }
            }
            if p != Nat.0 {
                pred(k.suc)
            }
        }
    }

    if p != Nat.0 {
        pred(fuel)
        let h: Bool =
            carry_high_fuel(p, a, b, c, fuel) =
                iterated_div(p, a + b + c, fuel)
        carry_high_fuel(p, a, b, c, fuel) =
            iterated_div(p, a + b + c, fuel)
    }
}

/// A carry process has no remaining high state once the fuel dominates the
/// whole sum.
theorem carry_high_fuel_zero_of_total_le_fuel(
    p: Nat, a: Nat, b: Nat, c: Nat, fuel: Nat
) {
    Nat.1 < p and a + b + c <= fuel implies
        carry_high_fuel(p, a, b, c, fuel) = Nat.0
} by {
    if Nat.1 < p and a + b + c <= fuel {
        p != Nat.0
        carry_high_fuel_eq_iterated_div_total(p, a, b, c, fuel)
        iterated_div_large(p, a + b + c, fuel)
        carry_high_fuel(p, a, b, c, fuel) = Nat.0
    }
}

/// If the fuel dominates the whole sum, then the left residual quotient has
/// vanished.
theorem left_residual_zero_of_total_le_fuel(
    p: Nat, a: Nat, b: Nat, c: Nat, fuel: Nat
) {
    Nat.1 < p and a + b + c <= fuel implies iterated_div(p, a, fuel) = Nat.0
} by {
    if Nat.1 < p and a + b + c <= fuel {
        carry_high_fuel_zero_of_total_le_fuel(p, a, b, c, fuel)
        carry_high_fuel_zero_imp_left_residual_zero(p, a, b, c, fuel)
        iterated_div(p, a, fuel) = Nat.0
    }
}

/// If the fuel dominates the whole sum, then the right residual quotient has
/// vanished.
theorem right_residual_zero_of_total_le_fuel(
    p: Nat, a: Nat, b: Nat, c: Nat, fuel: Nat
) {
    Nat.1 < p and a + b + c <= fuel implies iterated_div(p, b, fuel) = Nat.0
} by {
    if Nat.1 < p and a + b + c <= fuel {
        carry_high_fuel_zero_of_total_le_fuel(p, a, b, c, fuel)
        carry_high_fuel_zero_imp_right_residual_zero(p, a, b, c, fuel)
        iterated_div(p, b, fuel) = Nat.0
    }
}

/// If the fuel dominates the whole sum, then the residual incoming carry has
/// vanished.
theorem carry_residual_zero_of_total_le_fuel(
    p: Nat, a: Nat, b: Nat, c: Nat, fuel: Nat
) {
    Nat.1 < p and a + b + c <= fuel implies
        carry_in_fuel(p, a, b, c, fuel) = Nat.0
} by {
    if Nat.1 < p and a + b + c <= fuel {
        carry_high_fuel_zero_of_total_le_fuel(p, a, b, c, fuel)
        carry_high_fuel_zero_imp_carry_residual_zero(p, a, b, c, fuel)
        carry_in_fuel(p, a, b, c, fuel) = Nat.0
    }
}

/// The fuel `a + b + c` is enough for the carry process of `a + b + c` to
/// have no remaining high state.
theorem carry_high_fuel_zero_at_total(p: Nat, a: Nat, b: Nat, c: Nat) {
    Nat.1 < p implies carry_high_fuel(p, a, b, c, a + b + c) = Nat.0
} by {
    if Nat.1 < p {
        lte_ref(a + b + c)
        carry_high_fuel_zero_of_total_le_fuel(p, a, b, c, a + b + c)
        carry_high_fuel(p, a, b, c, a + b + c) = Nat.0
    }
}

/// The fuel `a + b` is enough for the carry process of `a + b` to have no
/// remaining high state.
theorem carry_high_fuel_zero_at_sum(p: Nat, a: Nat, b: Nat) {
    Nat.1 < p implies carry_high_fuel(p, a, b, Nat.0, a + b) = Nat.0
} by {
    if Nat.1 < p {
        lte_ref(a + b)
        carry_high_fuel_zero_of_total_le_fuel(p, a, b, Nat.0, a + b)
        carry_high_fuel(p, a, b, Nat.0, a + b) = Nat.0
    }
}

/// Once the fuel dominates the whole sum, the carry count satisfies the
/// digit-sum balance with incoming carry.
theorem carry_count_fuel_balance_of_total_le_fuel(
    p: Nat, a: Nat, b: Nat, c: Nat, fuel: Nat
) {
    Nat.1 < p and a + b + c <= fuel implies
        p * carry_count_fuel(p, a, b, c, fuel) + digit_sum(p, a + b + c) =
            carry_count_fuel(p, a, b, c, fuel) +
            digit_sum(p, a) + digit_sum(p, b) + c
} by {
    if Nat.1 < p and a + b + c <= fuel {
        carry_high_fuel_zero_of_total_le_fuel(p, a, b, c, fuel)
        carry_count_fuel_balance_of_high_zero(p, a, b, c, fuel)
        p * carry_count_fuel(p, a, b, c, fuel) + digit_sum(p, a + b + c) =
            carry_count_fuel(p, a, b, c, fuel) +
            digit_sum(p, a) + digit_sum(p, b) + c
    }
}

/// With fuel equal to the whole sum, the carry count satisfies the digit-sum
/// balance with incoming carry.
theorem carry_count_fuel_balance_at_total(p: Nat, a: Nat, b: Nat, c: Nat) {
    Nat.1 < p implies
        p * carry_count_fuel(p, a, b, c, a + b + c) + digit_sum(p, a + b + c) =
            carry_count_fuel(p, a, b, c, a + b + c) +
            digit_sum(p, a) + digit_sum(p, b) + c
} by {
    if Nat.1 < p {
        lte_ref(a + b + c)
        carry_count_fuel_balance_of_total_le_fuel(p, a, b, c, a + b + c)
        p * carry_count_fuel(p, a, b, c, a + b + c) +
            digit_sum(p, a + b + c) =
            carry_count_fuel(p, a, b, c, a + b + c) +
            digit_sum(p, a) + digit_sum(p, b) + c
    }
}

/// In a base greater than one, an equation of the form
/// `p * count + s = count + t` has at most one solution.
theorem carry_count_balance_unique(
    p: Nat, sum_digits: Nat, total_digits: Nat, count1: Nat, count2: Nat
) {
    Nat.1 < p and
    p * count1 + sum_digits = count1 + total_digits and
    p * count2 + sum_digits = count2 + total_digits
        implies count1 = count2
} by {
    if Nat.1 < p and
        p * count1 + sum_digits = count1 + total_digits and
        p * count2 + sum_digits = count2 + total_digits {
        lt_diff(Nat.1, p)
        let m: Nat satisfy { Nat.1 + m = p and m != Nat.0 }

        distrib_right(Nat.1, m, count1)
        mul_one_left(count1)
        add_cancels_left(count1, m * count1 + sum_digits, total_digits)
        m * count1 + sum_digits = total_digits

        distrib_right(Nat.1, m, count2)
        mul_one_left(count2)
        add_cancels_left(count2, m * count2 + sum_digits, total_digits)
        m * count2 + sum_digits = total_digits

        add_cancels_right(sum_digits, m * count1, m * count2)
        mul_cancel_left(m, count1, count2)
        count1 = count2
    }
}

/// True if `count` is the carry count for adding `a` and `b` with an
/// incoming carry `c`, as expressed by the digit-sum balance.
define is_carry_count_with_incoming(
    p: Nat, a: Nat, b: Nat, c: Nat, count: Nat
) -> Bool {
    p * count + digit_sum(p, a + b + c) =
        count + digit_sum(p, a) + digit_sum(p, b) + c
}

/// The zero-incoming carry-count predicate is the ordinary addition
/// carry-count predicate.
theorem is_carry_count_with_incoming_zero(
    p: Nat, a: Nat, b: Nat, count: Nat
) {
    is_carry_count_with_incoming(p, a, b, Nat.0, count) =
        is_addition_carry_count(p, a, b, count)
} by {
    a + b + Nat.0 = a + b
    count + digit_sum(p, a) + digit_sum(p, b) + Nat.0 =
        count + digit_sum(p, a) + digit_sum(p, b)
}

/// A zero-incoming carry count is an ordinary addition carry count.
theorem is_addition_carry_count_of_incoming_zero(
    p: Nat, a: Nat, b: Nat, count: Nat
) {
    is_carry_count_with_incoming(p, a, b, Nat.0, count) implies
        is_addition_carry_count(p, a, b, count)
} by {
    if is_carry_count_with_incoming(p, a, b, Nat.0, count) {
        is_carry_count_with_incoming_zero(p, a, b, count)
        is_addition_carry_count(p, a, b, count)
    }
}

/// An ordinary addition carry count is the zero-incoming carry count.
theorem incoming_zero_of_is_addition_carry_count(
    p: Nat, a: Nat, b: Nat, count: Nat
) {
    is_addition_carry_count(p, a, b, count) implies
        is_carry_count_with_incoming(p, a, b, Nat.0, count)
} by {
    if is_addition_carry_count(p, a, b, count) {
        is_carry_count_with_incoming_zero(p, a, b, count)
        is_carry_count_with_incoming(p, a, b, Nat.0, count)
    }
}

/// If the finite carry process has no remaining high state, then its carry
/// count satisfies the incoming-carry digit-sum predicate.
theorem carry_count_fuel_is_carry_count_with_incoming_of_high_zero(
    p: Nat, a: Nat, b: Nat, c: Nat, fuel: Nat
) {
    Nat.1 < p and carry_high_fuel(p, a, b, c, fuel) = Nat.0 implies
        is_carry_count_with_incoming(
            p, a, b, c, carry_count_fuel(p, a, b, c, fuel))
} by {
    if Nat.1 < p and carry_high_fuel(p, a, b, c, fuel) = Nat.0 {
        carry_count_fuel_balance_of_high_zero(p, a, b, c, fuel)
        is_carry_count_with_incoming(
            p, a, b, c, carry_count_fuel(p, a, b, c, fuel))
    }
}

/// If the fuel dominates the whole sum, then the finite carry count satisfies
/// the incoming-carry digit-sum predicate.
theorem carry_count_fuel_is_carry_count_with_incoming_of_total_le_fuel(
    p: Nat, a: Nat, b: Nat, c: Nat, fuel: Nat
) {
    Nat.1 < p and a + b + c <= fuel implies
        is_carry_count_with_incoming(
            p, a, b, c, carry_count_fuel(p, a, b, c, fuel))
} by {
    if Nat.1 < p and a + b + c <= fuel {
        carry_high_fuel_zero_of_total_le_fuel(p, a, b, c, fuel)
        carry_count_fuel_is_carry_count_with_incoming_of_high_zero(
            p, a, b, c, fuel)
        is_carry_count_with_incoming(
            p, a, b, c, carry_count_fuel(p, a, b, c, fuel))
    }
}

/// With fuel equal to the whole sum, the finite carry count satisfies the
/// incoming-carry digit-sum predicate.
theorem carry_count_fuel_is_carry_count_with_incoming_at_total(
    p: Nat, a: Nat, b: Nat, c: Nat
) {
    Nat.1 < p implies
        is_carry_count_with_incoming(
            p, a, b, c, carry_count_fuel(p, a, b, c, a + b + c))
} by {
    if Nat.1 < p {
        lte_ref(a + b + c)
        carry_count_fuel_is_carry_count_with_incoming_of_total_le_fuel(
            p, a, b, c, a + b + c)
        is_carry_count_with_incoming(
            p, a, b, c, carry_count_fuel(p, a, b, c, a + b + c))
    }
}

/// In a base greater than one, the incoming-carry digit-sum equation has at
/// most one solution.
theorem is_carry_count_with_incoming_unique(
    p: Nat, a: Nat, b: Nat, c: Nat, count1: Nat, count2: Nat
) {
    Nat.1 < p and
    is_carry_count_with_incoming(p, a, b, c, count1) and
    is_carry_count_with_incoming(p, a, b, c, count2)
        implies count1 = count2
} by {
    if Nat.1 < p and
        is_carry_count_with_incoming(p, a, b, c, count1) and
        is_carry_count_with_incoming(p, a, b, c, count2) {
        let sum_digits: Nat = digit_sum(p, a + b + c)
        let total_digits: Nat = digit_sum(p, a) + digit_sum(p, b) + c
        p * count1 + digit_sum(p, a + b + c) =
            count1 + digit_sum(p, a) + digit_sum(p, b) + c
        p * count2 + digit_sum(p, a + b + c) =
            count2 + digit_sum(p, a) + digit_sum(p, b) + c
        p * count1 + sum_digits = count1 + total_digits
        p * count2 + sum_digits = count2 + total_digits
        carry_count_balance_unique(p, sum_digits, total_digits, count1, count2)
        count1 = count2
    }
}

/// Any solution of the incoming-carry digit-sum equation is the finite carry
/// count, once the finite process has no remaining high state.
theorem carry_count_fuel_eq_of_is_carry_count_with_incoming_of_high_zero(
    p: Nat, a: Nat, b: Nat, c: Nat, fuel: Nat, count: Nat
) {
    Nat.1 < p and
    carry_high_fuel(p, a, b, c, fuel) = Nat.0 and
    is_carry_count_with_incoming(p, a, b, c, count)
        implies carry_count_fuel(p, a, b, c, fuel) = count
} by {
    if Nat.1 < p and
        carry_high_fuel(p, a, b, c, fuel) = Nat.0 and
        is_carry_count_with_incoming(p, a, b, c, count) {
        carry_count_fuel_is_carry_count_with_incoming_of_high_zero(
            p, a, b, c, fuel)
        is_carry_count_with_incoming_unique(
            p, a, b, c, carry_count_fuel(p, a, b, c, fuel), count)
        carry_count_fuel(p, a, b, c, fuel) = count
    }
}

/// Any solution of the incoming-carry digit-sum equation is equal to the
/// finite carry count, once the finite process has no remaining high state.
theorem is_carry_count_with_incoming_eq_carry_count_fuel_of_high_zero(
    p: Nat, a: Nat, b: Nat, c: Nat, fuel: Nat, count: Nat
) {
    Nat.1 < p and
    carry_high_fuel(p, a, b, c, fuel) = Nat.0 and
    is_carry_count_with_incoming(p, a, b, c, count)
        implies count = carry_count_fuel(p, a, b, c, fuel)
} by {
    if Nat.1 < p and
        carry_high_fuel(p, a, b, c, fuel) = Nat.0 and
        is_carry_count_with_incoming(p, a, b, c, count) {
        carry_count_fuel_eq_of_is_carry_count_with_incoming_of_high_zero(
            p, a, b, c, fuel, count)
        count = carry_count_fuel(p, a, b, c, fuel)
    }
}

/// Any solution of the incoming-carry digit-sum equation is the finite carry
/// count for every sufficiently large fuel.
theorem carry_count_fuel_eq_of_is_carry_count_with_incoming_of_total_le_fuel(
    p: Nat, a: Nat, b: Nat, c: Nat, fuel: Nat, count: Nat
) {
    Nat.1 < p and
    a + b + c <= fuel and
    is_carry_count_with_incoming(p, a, b, c, count)
        implies carry_count_fuel(p, a, b, c, fuel) = count
} by {
    if Nat.1 < p and
        a + b + c <= fuel and
        is_carry_count_with_incoming(p, a, b, c, count) {
        carry_high_fuel_zero_of_total_le_fuel(p, a, b, c, fuel)
        carry_count_fuel_eq_of_is_carry_count_with_incoming_of_high_zero(
            p, a, b, c, fuel, count)
        carry_count_fuel(p, a, b, c, fuel) = count
    }
}

/// Any solution of the incoming-carry digit-sum equation is equal to the
/// finite carry count for every sufficiently large fuel.
theorem is_carry_count_with_incoming_eq_carry_count_fuel_of_total_le_fuel(
    p: Nat, a: Nat, b: Nat, c: Nat, fuel: Nat, count: Nat
) {
    Nat.1 < p and
    a + b + c <= fuel and
    is_carry_count_with_incoming(p, a, b, c, count)
        implies count = carry_count_fuel(p, a, b, c, fuel)
} by {
    if Nat.1 < p and
        a + b + c <= fuel and
        is_carry_count_with_incoming(p, a, b, c, count) {
        carry_count_fuel_eq_of_is_carry_count_with_incoming_of_total_le_fuel(
            p, a, b, c, fuel, count)
        count = carry_count_fuel(p, a, b, c, fuel)
    }
}

/// Two sufficiently large fuels give the same incoming-carry count.
theorem carry_count_fuel_eq_of_total_le_fuels(
    p: Nat, a: Nat, b: Nat, c: Nat, fuel1: Nat, fuel2: Nat
) {
    Nat.1 < p and a + b + c <= fuel1 and a + b + c <= fuel2 implies
        carry_count_fuel(p, a, b, c, fuel1) =
            carry_count_fuel(p, a, b, c, fuel2)
} by {
    if Nat.1 < p and a + b + c <= fuel1 and a + b + c <= fuel2 {
        carry_count_fuel_is_carry_count_with_incoming_of_total_le_fuel(
            p, a, b, c, fuel1)
        carry_count_fuel_is_carry_count_with_incoming_of_total_le_fuel(
            p, a, b, c, fuel2)
        is_carry_count_with_incoming(
            p, a, b, c, carry_count_fuel(p, a, b, c, fuel2))
        is_carry_count_with_incoming_unique(
            p, a, b, c,
            carry_count_fuel(p, a, b, c, fuel1),
            carry_count_fuel(p, a, b, c, fuel2))
        carry_count_fuel(p, a, b, c, fuel1) =
            carry_count_fuel(p, a, b, c, fuel2)
    }
}

/// Every sufficiently large fuel gives the same incoming-carry count as the
/// canonical fuel `a + b + c`.
theorem carry_count_fuel_eq_at_total_of_total_le_fuel(
    p: Nat, a: Nat, b: Nat, c: Nat, fuel: Nat
) {
    Nat.1 < p and a + b + c <= fuel implies
        carry_count_fuel(p, a, b, c, fuel) =
            carry_count_fuel(p, a, b, c, a + b + c)
} by {
    if Nat.1 < p and a + b + c <= fuel {
        lte_ref(a + b + c)
        carry_count_fuel_eq_of_total_le_fuels(
            p, a, b, c, fuel, a + b + c)
        carry_count_fuel(p, a, b, c, fuel) =
            carry_count_fuel(p, a, b, c, a + b + c)
    }
}

/// Any fuel that dominates `a + b` makes the recursive carry count satisfy the
/// carry-count predicate for `a + b`.
theorem carry_count_fuel_is_addition_carry_count_of_sum_le_fuel(
    p: Nat, a: Nat, b: Nat, fuel: Nat
) {
    Nat.1 < p and a + b <= fuel implies
        is_addition_carry_count(
            p, a, b, carry_count_fuel(p, a, b, Nat.0, fuel))
} by {
    if Nat.1 < p and a + b <= fuel {
        carry_high_fuel_zero_of_total_le_fuel(p, a, b, Nat.0, fuel)
        carry_count_fuel_is_addition_carry_count(p, a, b, fuel)
        is_addition_carry_count(
            p, a, b, carry_count_fuel(p, a, b, Nat.0, fuel))
    }
}

/// With fuel equal to `a + b`, the recursive carry count satisfies the
/// carry-count predicate for `a + b`.
theorem carry_count_fuel_is_addition_carry_count_at_sum(p: Nat, a: Nat, b: Nat) {
    Nat.1 < p implies
        is_addition_carry_count(
            p, a, b, carry_count_fuel(p, a, b, Nat.0, a + b))
} by {
    if Nat.1 < p {
        lte_ref(a + b)
        carry_count_fuel_is_addition_carry_count_of_sum_le_fuel(p, a, b, a + b)
        is_addition_carry_count(
            p, a, b, carry_count_fuel(p, a, b, Nat.0, a + b))
    }
}

/// The recursive carry count for adding `a` and `b` in base `p`.
define addition_carry_count(p: Nat, a: Nat, b: Nat) -> Nat {
    carry_count_fuel(p, a, b, Nat.0, a + b)
}

/// The recursive carry count unfolds to the enough-fuel carry count.
theorem addition_carry_count_eq(p: Nat, a: Nat, b: Nat) {
    addition_carry_count(p, a, b) = carry_count_fuel(p, a, b, Nat.0, a + b)
}

/// The recursive carry count satisfies the carry-count predicate.
theorem addition_carry_count_is_addition_carry_count(p: Nat, a: Nat, b: Nat) {
    Nat.1 < p implies is_addition_carry_count(p, a, b, addition_carry_count(p, a, b))
} by {
    if Nat.1 < p {
        addition_carry_count_eq(p, a, b)
        carry_count_fuel_is_addition_carry_count_of_sum_le_fuel(p, a, b, a + b)
        is_addition_carry_count(
            p, a, b, carry_count_fuel(p, a, b, Nat.0, a + b))
        is_addition_carry_count(p, a, b, addition_carry_count(p, a, b))
    }
}

/// In a base greater than one, the digit-sum carry-count equation has at most
/// one solution.
theorem is_addition_carry_count_unique(
    p: Nat, a: Nat, b: Nat, count1: Nat, count2: Nat
) {
    Nat.1 < p and
    is_addition_carry_count(p, a, b, count1) and
    is_addition_carry_count(p, a, b, count2)
        implies count1 = count2
} by {
    if Nat.1 < p and
        is_addition_carry_count(p, a, b, count1) and
        is_addition_carry_count(p, a, b, count2) {
        let total_digits: Nat = digit_sum(p, a) + digit_sum(p, b)
        let sum_digits: Nat = digit_sum(p, a + b)

        is_addition_carry_count(p, a, b, count1) =
            (p * count1 + sum_digits = count1 + total_digits)
        is_addition_carry_count(p, a, b, count2) =
            (p * count2 + sum_digits = count2 + total_digits)
        carry_count_balance_unique(p, sum_digits, total_digits, count1, count2)
        count1 = count2
    }
}

/// Any solution of the digit-sum carry-count equation is the recursive carry
/// count.
theorem addition_carry_count_eq_of_is_addition_carry_count(
    p: Nat, a: Nat, b: Nat, count: Nat
) {
    Nat.1 < p and is_addition_carry_count(p, a, b, count)
        implies addition_carry_count(p, a, b) = count
} by {
    if Nat.1 < p and is_addition_carry_count(p, a, b, count) {
        addition_carry_count_is_addition_carry_count(p, a, b)
        is_addition_carry_count_unique(
            p, a, b, addition_carry_count(p, a, b), count)
        addition_carry_count(p, a, b) = count
    }
}

/// Any solution of the digit-sum carry-count equation is equal to the
/// recursive carry count.
theorem is_addition_carry_count_eq_addition_carry_count(
    p: Nat, a: Nat, b: Nat, count: Nat
) {
    Nat.1 < p and is_addition_carry_count(p, a, b, count)
        implies count = addition_carry_count(p, a, b)
} by {
    if Nat.1 < p and is_addition_carry_count(p, a, b, count) {
        addition_carry_count_eq_of_is_addition_carry_count(p, a, b, count)
        count = addition_carry_count(p, a, b)
    }
}

/// Any sufficiently large zero-incoming fuel gives the recursive carry count.
theorem carry_count_fuel_eq_addition_carry_count_of_sum_le_fuel(
    p: Nat, a: Nat, b: Nat, fuel: Nat
) {
    Nat.1 < p and a + b <= fuel implies
        carry_count_fuel(p, a, b, Nat.0, fuel) =
            addition_carry_count(p, a, b)
} by {
    if Nat.1 < p and a + b <= fuel {
        carry_count_fuel_is_addition_carry_count_of_sum_le_fuel(
            p, a, b, fuel)
        is_addition_carry_count_eq_addition_carry_count(
            p, a, b, carry_count_fuel(p, a, b, Nat.0, fuel))
        carry_count_fuel(p, a, b, Nat.0, fuel) =
            addition_carry_count(p, a, b)
    }
}

/// The recursive carry count is equal to any sufficiently large
/// zero-incoming fuel count.
theorem addition_carry_count_eq_carry_count_fuel_of_sum_le_fuel(
    p: Nat, a: Nat, b: Nat, fuel: Nat
) {
    Nat.1 < p and a + b <= fuel implies
        addition_carry_count(p, a, b) =
            carry_count_fuel(p, a, b, Nat.0, fuel)
} by {
    if Nat.1 < p and a + b <= fuel {
        carry_count_fuel_eq_addition_carry_count_of_sum_le_fuel(
            p, a, b, fuel)
        addition_carry_count(p, a, b) =
            carry_count_fuel(p, a, b, Nat.0, fuel)
    }
}

/// Any zero-incoming fuel whose high state has vanished gives the recursive
/// carry count.
theorem carry_count_fuel_eq_addition_carry_count_of_high_zero(
    p: Nat, a: Nat, b: Nat, fuel: Nat
) {
    Nat.1 < p and carry_high_fuel(p, a, b, Nat.0, fuel) = Nat.0 implies
        carry_count_fuel(p, a, b, Nat.0, fuel) =
            addition_carry_count(p, a, b)
} by {
    if Nat.1 < p and carry_high_fuel(p, a, b, Nat.0, fuel) = Nat.0 {
        carry_count_fuel_is_addition_carry_count(p, a, b, fuel)
        is_addition_carry_count_eq_addition_carry_count(
            p, a, b, carry_count_fuel(p, a, b, Nat.0, fuel))
        carry_count_fuel(p, a, b, Nat.0, fuel) =
            addition_carry_count(p, a, b)
    }
}

/// The recursive carry count is equal to any zero-incoming fuel count whose
/// high state has vanished.
theorem addition_carry_count_eq_carry_count_fuel_of_high_zero(
    p: Nat, a: Nat, b: Nat, fuel: Nat
) {
    Nat.1 < p and carry_high_fuel(p, a, b, Nat.0, fuel) = Nat.0 implies
        addition_carry_count(p, a, b) =
            carry_count_fuel(p, a, b, Nat.0, fuel)
} by {
    if Nat.1 < p and carry_high_fuel(p, a, b, Nat.0, fuel) = Nat.0 {
        carry_count_fuel_eq_addition_carry_count_of_high_zero(
            p, a, b, fuel)
        addition_carry_count(p, a, b) =
            carry_count_fuel(p, a, b, Nat.0, fuel)
    }
}

/// Any sufficiently large zero-incoming fuel gives the same carry count as the
/// canonical fuel `a + b`.
theorem carry_count_fuel_eq_at_sum_of_sum_le_fuel(
    p: Nat, a: Nat, b: Nat, fuel: Nat
) {
    Nat.1 < p and a + b <= fuel implies
        carry_count_fuel(p, a, b, Nat.0, fuel) =
            carry_count_fuel(p, a, b, Nat.0, a + b)
} by {
    if Nat.1 < p and a + b <= fuel {
        carry_count_fuel_eq_addition_carry_count_of_sum_le_fuel(
            p, a, b, fuel)
        addition_carry_count_eq(p, a, b)
        carry_count_fuel(p, a, b, Nat.0, fuel) =
            carry_count_fuel(p, a, b, Nat.0, a + b)
    }
}

/// The canonical fuel `a + b` gives the same carry count as any sufficiently
/// large zero-incoming fuel.
theorem carry_count_fuel_at_sum_eq_of_sum_le_fuel(
    p: Nat, a: Nat, b: Nat, fuel: Nat
) {
    Nat.1 < p and a + b <= fuel implies
        carry_count_fuel(p, a, b, Nat.0, a + b) =
            carry_count_fuel(p, a, b, Nat.0, fuel)
} by {
    if Nat.1 < p and a + b <= fuel {
        carry_count_fuel_eq_at_sum_of_sum_le_fuel(p, a, b, fuel)
        carry_count_fuel(p, a, b, Nat.0, a + b) =
            carry_count_fuel(p, a, b, Nat.0, fuel)
    }
}

/// Any zero-incoming fuel whose high state has vanished gives the same carry
/// count as the canonical fuel `a + b`.
theorem carry_count_fuel_eq_at_sum_of_high_zero(
    p: Nat, a: Nat, b: Nat, fuel: Nat
) {
    Nat.1 < p and carry_high_fuel(p, a, b, Nat.0, fuel) = Nat.0 implies
        carry_count_fuel(p, a, b, Nat.0, fuel) =
            carry_count_fuel(p, a, b, Nat.0, a + b)
} by {
    if Nat.1 < p and carry_high_fuel(p, a, b, Nat.0, fuel) = Nat.0 {
        carry_count_fuel_eq_addition_carry_count_of_high_zero(
            p, a, b, fuel)
        addition_carry_count_eq(p, a, b)
        carry_count_fuel(p, a, b, Nat.0, fuel) =
            carry_count_fuel(p, a, b, Nat.0, a + b)
    }
}

/// The canonical fuel `a + b` gives the same carry count as any zero-incoming
/// fuel whose high state has vanished.
theorem carry_count_fuel_at_sum_eq_of_high_zero(
    p: Nat, a: Nat, b: Nat, fuel: Nat
) {
    Nat.1 < p and carry_high_fuel(p, a, b, Nat.0, fuel) = Nat.0 implies
        carry_count_fuel(p, a, b, Nat.0, a + b) =
            carry_count_fuel(p, a, b, Nat.0, fuel)
} by {
    if Nat.1 < p and carry_high_fuel(p, a, b, Nat.0, fuel) = Nat.0 {
        carry_count_fuel_eq_at_sum_of_high_zero(p, a, b, fuel)
        carry_count_fuel(p, a, b, Nat.0, a + b) =
            carry_count_fuel(p, a, b, Nat.0, fuel)
    }
}

/// The recursive carry count is symmetric in the two addends.
theorem addition_carry_count_comm(p: Nat, a: Nat, b: Nat) {
    addition_carry_count(p, a, b) = addition_carry_count(p, b, a)
} by {
    addition_carry_count_eq(p, a, b)
    addition_carry_count_eq(p, b, a)
    add_comm(a, b)
    carry_count_fuel_comm(p, a, b, Nat.0, a + b)
}

/// Adding zero on the right gives recursive carry count zero.
theorem addition_carry_count_zero_right(p: Nat, a: Nat) {
    p != Nat.0 implies addition_carry_count(p, a, Nat.0) = Nat.0
} by {
    if p != Nat.0 {
        addition_carry_count_eq(p, a, Nat.0)
        carry_count_fuel_zero_right(p, a, a)
        addition_carry_count(p, a, Nat.0) = Nat.0
    }
}

/// Adding zero on the left gives recursive carry count zero.
theorem addition_carry_count_zero_left(p: Nat, b: Nat) {
    p != Nat.0 implies addition_carry_count(p, Nat.0, b) = Nat.0
} by {
    if p != Nat.0 {
        addition_carry_count_comm(p, Nat.0, b)
        addition_carry_count_zero_right(p, b)
        addition_carry_count(p, Nat.0, b) = Nat.0
    }
}

/// The carry count for adding a natural number to itself in base `p`.
define double_addition_carry_count(p: Nat, n: Nat) -> Nat {
    addition_carry_count(p, n, n)
}

/// The doubled-addend carry count unfolds to the ordinary recursive carry
/// count.
theorem double_addition_carry_count_eq(p: Nat, n: Nat) {
    double_addition_carry_count(p, n) = addition_carry_count(p, n, n)
}

/// The doubled-addend carry count satisfies the carry-count predicate.
theorem double_addition_carry_count_is_addition_carry_count(p: Nat, n: Nat) {
    Nat.1 < p implies
        is_addition_carry_count(p, n, n, double_addition_carry_count(p, n))
} by {
    if Nat.1 < p {
        double_addition_carry_count_eq(p, n)
        addition_carry_count_is_addition_carry_count(p, n, n)
        is_addition_carry_count(p, n, n, double_addition_carry_count(p, n))
    }
}

/// Any sufficiently large zero-incoming fuel for doubling gives the
/// doubled-addend recursive carry count.
theorem carry_count_fuel_eq_double_addition_carry_count_of_double_le_fuel(
    p: Nat, n: Nat, fuel: Nat
) {
    Nat.1 < p and n + n <= fuel implies
        carry_count_fuel(p, n, n, Nat.0, fuel) =
            double_addition_carry_count(p, n)
} by {
    if Nat.1 < p and n + n <= fuel {
        carry_count_fuel_eq_addition_carry_count_of_sum_le_fuel(
            p, n, n, fuel)
        double_addition_carry_count_eq(p, n)
        carry_count_fuel(p, n, n, Nat.0, fuel) =
            double_addition_carry_count(p, n)
    }
}

/// The doubled-addend recursive carry count is equal to any sufficiently
/// large zero-incoming fuel count for doubling.
theorem double_addition_carry_count_eq_carry_count_fuel_of_double_le_fuel(
    p: Nat, n: Nat, fuel: Nat
) {
    Nat.1 < p and n + n <= fuel implies
        double_addition_carry_count(p, n) =
            carry_count_fuel(p, n, n, Nat.0, fuel)
} by {
    if Nat.1 < p and n + n <= fuel {
        carry_count_fuel_eq_double_addition_carry_count_of_double_le_fuel(
            p, n, fuel)
        double_addition_carry_count(p, n) =
            carry_count_fuel(p, n, n, Nat.0, fuel)
    }
}

/// Any sufficiently large zero-incoming fuel for doubling gives the same
/// carry count as the canonical fuel `n + n`.
theorem carry_count_fuel_eq_at_double_sum_of_double_le_fuel(
    p: Nat, n: Nat, fuel: Nat
) {
    Nat.1 < p and n + n <= fuel implies
        carry_count_fuel(p, n, n, Nat.0, fuel) =
            carry_count_fuel(p, n, n, Nat.0, n + n)
} by {
    if Nat.1 < p and n + n <= fuel {
        carry_count_fuel_eq_at_sum_of_sum_le_fuel(p, n, n, fuel)
        carry_count_fuel(p, n, n, Nat.0, fuel) =
            carry_count_fuel(p, n, n, Nat.0, n + n)
    }
}

/// The canonical fuel `n + n` gives the same carry count as any sufficiently
/// large zero-incoming fuel for doubling.
theorem carry_count_fuel_at_double_sum_eq_of_double_le_fuel(
    p: Nat, n: Nat, fuel: Nat
) {
    Nat.1 < p and n + n <= fuel implies
        carry_count_fuel(p, n, n, Nat.0, n + n) =
            carry_count_fuel(p, n, n, Nat.0, fuel)
} by {
    if Nat.1 < p and n + n <= fuel {
        carry_count_fuel_eq_at_double_sum_of_double_le_fuel(p, n, fuel)
        carry_count_fuel(p, n, n, Nat.0, n + n) =
            carry_count_fuel(p, n, n, Nat.0, fuel)
    }
}

/// Adding zero to itself gives doubled-addend carry count zero.
theorem double_addition_carry_count_zero(p: Nat) {
    p != Nat.0 implies double_addition_carry_count(p, Nat.0) = Nat.0
} by {
    if p != Nat.0 {
        double_addition_carry_count_eq(p, Nat.0)
        addition_carry_count_zero_right(p, Nat.0)
        double_addition_carry_count(p, Nat.0) = Nat.0
    }
}
