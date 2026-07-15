from nat.nat_base import Nat, alt_suc_ne_zero, add_zero_left, add_zero_right,
    mul_zero_left, mul_one_right, small_mod, div_imp_mod, zero_divides,
    only_zero_lte_zero, lt_imp_lte_suc, lte_cancel_suc, divides_self,
    divides_symm
from nat.division import div_mod_decomp, mod_lt, div_of_decomp, mod_of_decomp,
    div_mul
from nat.count_multiples import count_multiples, count_multiples_eq_div, zero_div
from nat.digit_sum import digit_sum, digit_sum_fuel, digit_sum_zero,
    digit_sum_fuel_at_zero, digit_sum_fuel_step, digit_sum_recurrence
from nat.nat_monoid import exp_add, exp_mul, exp_ne_zero, sq_eq_mul
from nat.gcd import gcd_divides_left, gcd_divides_right, divides_gcd, gcd_comm,
    gcd_nonzero_left
from nat.lcm import gcd_mul_lcm
numerals Nat

/// Symmetric form of the division/modulus decomposition.
theorem div_mod_decomp_eq(a: Nat, m: Nat) {
    a = a.div(m) * m + a.mod(m)
} by {
    div_mod_decomp(a, m)
}

/// The quotient is unique for a decomposition with a small remainder.
theorem div_of_decomp_eq(a: Nat, q: Nat, r: Nat, m: Nat) {
    r < m and a = q * m + r implies a.div(m) = q
} by {
    if r < m and a = q * m + r {
        div_of_decomp(q, r, m)
        a.div(m) = q
    }
}

/// The remainder is unique for a decomposition with a small remainder.
theorem mod_of_decomp_eq(a: Nat, q: Nat, r: Nat, m: Nat) {
    r < m and a = q * m + r implies a.mod(m) = r
} by {
    if r < m and a = q * m + r {
        mod_of_decomp(q, r, m)
        a.mod(m) = r
    }
}

/// Division by one is the identity.
theorem div_one(n: Nat) {
    n.div(Nat.1) = n
} by {
    div_mul(n, Nat.1)
    (n * Nat.1).div(Nat.1) = n
    mul_one_right(n)
}

/// Modulo one is always zero.
theorem mod_one(n: Nat) {
    n.mod(Nat.1) = Nat.0
} by {
    mod_lt(n, Nat.1)
    n.mod(Nat.1) < Nat.1
    lt_imp_lte_suc(n.mod(Nat.1), Nat.0)
    lte_cancel_suc(n.mod(Nat.1), Nat.0)
    only_zero_lte_zero(n.mod(Nat.1))
}

/// If `m` divides `a`, division by `m` reconstructs `a` without a remainder.
theorem div_mul_of_divides(a: Nat, m: Nat) {
    m != Nat.0 and m.divides(a) implies a.div(m) * m = a
} by {
    if m != Nat.0 and m.divides(a) {
        div_imp_mod(a, m)
        div_mod_decomp(a, m)
        add_zero_right(a.div(m) * m)
        a.div(m) * m = a
    }
}

/// The count of multiples in the empty range is zero.
theorem count_multiples_zero_right(d: Nat) {
    count_multiples(d, Nat.0) = Nat.0
}

/// Counting multiples steps up when the new endpoint is divisible.
theorem count_multiples_suc_of_divides(d: Nat, n: Nat) {
    d.divides(n.suc) implies
        count_multiples(d, n.suc) = count_multiples(d, n) + Nat.1
}

/// Counting multiples is unchanged when the new endpoint is not divisible.
theorem count_multiples_suc_of_not_divides(d: Nat, n: Nat) {
    not d.divides(n.suc) implies count_multiples(d, n.suc) = count_multiples(d, n)
}

/// Counting multiples of a nonzero divisor agrees with quotient.
theorem count_multiples_eq_div_nonzero(d: Nat, n: Nat) {
    d != Nat.0 implies count_multiples(d, n) = n.div(d)
} by {
    if d != Nat.0 {
        count_multiples_eq_div(d, n)
    }
}

/// Every positive endpoint is a multiple of one, so the count is the endpoint.
theorem count_multiples_one(n: Nat) {
    count_multiples(Nat.1, n) = n
} by {
    count_multiples_eq_div(Nat.1, n)
    count_multiples(Nat.1, n) = n.div(Nat.1)
    div_one(n)
}

/// Zero has no positive multiples in the range `1..=n`.
theorem count_multiples_zero_left(n: Nat) {
    count_multiples(Nat.0, n) = Nat.0
} by {
    define p(k: Nat) -> Bool {
        count_multiples(Nat.0, k) = Nat.0
    }
    p(Nat.0)
    forall(k: Nat) {
        if p(k) {
            if Nat.0.divides(k.suc) {
                zero_divides(k.suc)
                alt_suc_ne_zero(k)
                false
            }
            count_multiples(Nat.0, k.suc) = count_multiples(Nat.0, k)
            p(k.suc)
        }
    }
    p(n)
}

/// Existing `zero_div` under a descriptive name.
theorem zero_div_by_nonzero(d: Nat) {
    d != Nat.0 implies Nat.0.div(d) = Nat.0
} by {
    if d != Nat.0 {
        zero_div(d)
    }
}

/// The digit sum of zero vanishes for every explicit recursion budget.
theorem digit_sum_fuel_zero(p: Nat, fuel: Nat) {
    digit_sum_fuel(p, Nat.0, fuel) = Nat.0
} by {
    digit_sum_fuel_at_zero(p, fuel)
}

/// One step of the explicit digit-sum recursion, as a named client endpoint.
theorem digit_sum_fuel_suc_nonzero(p: Nat, n: Nat, fuel: Nat) {
    n != Nat.0 implies
        digit_sum_fuel(p, n, fuel.suc) = n.mod(p) + digit_sum_fuel(p, n.div(p), fuel)
} by {
    if n != Nat.0 {
        digit_sum_fuel_step(p, n, fuel)
    }
}

/// In a base larger than `n`, the digit sum of `n` is just `n`.
theorem digit_sum_small(p: Nat, n: Nat) {
    Nat.1 < p and n < p implies digit_sum(p, n) = n
} by {
    if Nat.1 < p and n < p {
        if n = Nat.0 {
            digit_sum_zero(p)
            digit_sum(p, n) = n
        }
        if n != Nat.0 {
            digit_sum_recurrence(p, n)
            small_mod(n, p)
            div_of_decomp(Nat.0, n, p)
            mul_zero_left(p)
            add_zero_left(n)
            digit_sum_zero(p)
            digit_sum(p, n) = n
        }
    }
}

/// Nonzero-base powers are nonzero.
theorem nat_pow_nonzero_of_base_nonzero(a: Nat, n: Nat) {
    a != Nat.0 implies a.pow(n) != Nat.0
} by {
    if a != Nat.0 {
        exp_ne_zero(a, n)
    }
}

/// The successor exponent unfolds one multiplication on the left.
theorem nat_pow_suc(a: Nat, n: Nat) {
    a.pow(n.suc) = a * a.pow(n)
}

/// The square power is multiplication by itself.
theorem nat_pow_two(a: Nat) {
    a.pow(Nat.2) = a * a
} by {
    sq_eq_mul(a)
}

/// Power of a sum of exponents, oriented from product to a single power.
theorem nat_pow_add_product(a: Nat, m: Nat, n: Nat) {
    a.pow(m) * a.pow(n) = a.pow(m + n)
} by {
    exp_add(a, m, n)
}

/// Power of a product of exponents, oriented toward a power of a power.
theorem nat_pow_mul_power(a: Nat, m: Nat, n: Nat) {
    a.pow(m * n) = a.pow(m).pow(n)
} by {
    exp_mul(a, m, n)
}

/// The gcd of a natural with itself is itself.
theorem nat_gcd_self(a: Nat) {
    a.gcd(a) = a
} by {
    gcd_divides_left(a, a)
    divides_self(a)
    divides_symm(a.gcd(a), a)
}

/// The gcd divides both inputs, packaged as a conjunction.
theorem nat_gcd_divides_both(a: Nat, b: Nat) {
    a.gcd(b).divides(a) and a.gcd(b).divides(b)
} by {
    gcd_divides_left(a, b)
    gcd_divides_right(a, b)
}

/// Any common divisor divides the gcd.
theorem nat_common_divisor_divides_gcd(a: Nat, b: Nat, d: Nat) {
    d.divides(a) and d.divides(b) implies d.divides(a.gcd(b))
} by {
    if d.divides(a) and d.divides(b) {
        divides_gcd(d, a, b)
    }
}

/// A gcd is zero only when both inputs are zero.
theorem nat_gcd_eq_zero_imp_inputs_zero(a: Nat, b: Nat) {
    a.gcd(b) = Nat.0 implies a = Nat.0 and b = Nat.0
} by {
    if a.gcd(b) = Nat.0 {
        if a != Nat.0 {
            gcd_nonzero_left(a, b)
            false
        }
        if b != Nat.0 {
            gcd_comm(a, b)
            gcd_nonzero_left(b, a)
            false
        }
        b = Nat.0
    }
}

/// Zero is absorbing for lcm on the left.
theorem nat_lcm_zero_left(a: Nat) {
    Nat.0.lcm(a) = Nat.0
} by {
    if a = Nat.0 {
        Nat.0.gcd(a) = Nat.0
    } else {
        Nat.0 * a = Nat.0
        a * Nat.0.lcm(a) = Nat.0
        Nat.0.lcm(a) = Nat.0
    }
}

/// Zero is absorbing for lcm on the right.
theorem nat_lcm_zero_right(a: Nat) {
    a.lcm(Nat.0) = Nat.0
} by {
    if a = Nat.0 {
        a.gcd(Nat.0) = Nat.0
    } else {
        a * Nat.0 = Nat.0
        a * a.lcm(Nat.0) = Nat.0
        a.lcm(Nat.0) = Nat.0
    }
}

/// The least common multiple is symmetric.
theorem nat_lcm_comm(a: Nat, b: Nat) {
    a.lcm(b) = b.lcm(a)
} by {
    gcd_comm(a, b)
    a.gcd(b) = b.gcd(a)
    gcd_mul_lcm(a, b)
    gcd_mul_lcm(b, a)
    a.gcd(b) * a.lcm(b) = a * b
    b.gcd(a) * b.lcm(a) = b * a
    b * a = a * b
    a.gcd(b) * b.lcm(a) = a * b
    a.gcd(b) * a.lcm(b) = a.gcd(b) * b.lcm(a)
    if a.gcd(b) = Nat.0 {
        gcd_nonzero_left(a, b)
        gcd_comm(a, b)
        gcd_nonzero_left(b, a)
        nat_lcm_zero_left(b)
        nat_lcm_zero_right(b)
        b.lcm(a) = Nat.0
    } else {
        a.lcm(b) = b.lcm(a)
    }
}

/// The lcm of a natural with itself is itself.
theorem nat_lcm_self(a: Nat) {
    a.lcm(a) = a
} by {
    if a = Nat.0 {
        nat_lcm_zero_left(Nat.0)
        a.lcm(a) = a
    } else {
        nat_gcd_self(a)
        a.gcd(a) = a
        gcd_mul_lcm(a, a)
        a.gcd(a) * a.lcm(a) = a * a
        a * a.lcm(a) = a * a
        a.lcm(a) = a
    }
}
