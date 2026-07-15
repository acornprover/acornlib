from nat.lattice import Nat

numerals Nat

/// Small addition facts used by the one-digit multiplication table.
theorem nat_add_0_1 {
    0 + 1 = 1
} by {
}

theorem nat_add_0_2 {
    0 + 2 = 2
} by {
    nat_add_0_1
}

theorem nat_add_0_3 {
    0 + 3 = 3
} by {
    nat_add_0_2
}

theorem nat_add_0_4 {
    0 + 4 = 4
} by {
    nat_add_0_3
}

theorem nat_add_0_5 {
    0 + 5 = 5
} by {
    nat_add_0_4
}

theorem nat_add_0_6 {
    0 + 6 = 6
} by {
    nat_add_0_5
}

theorem nat_add_0_7 {
    0 + 7 = 7
} by {
    nat_add_0_6
}

theorem nat_add_0_8 {
    0 + 8 = 8
} by {
    nat_add_0_7
}

theorem nat_add_0_9 {
    0 + 9 = 9
} by {
    nat_add_0_8
}

theorem nat_add_1_1 {
    1 + 1 = 2
} by {
}

theorem nat_add_2_1 {
    2 + 1 = 3
} by {
}

theorem nat_add_2_2 {
    2 + 2 = 4
} by {
    nat_add_2_1
}

theorem nat_add_3_1 {
    3 + 1 = 4
} by {
}

theorem nat_add_3_2 {
    3 + 2 = 5
} by {
    nat_add_3_1
}

theorem nat_add_3_3 {
    3 + 3 = 6
} by {
    nat_add_3_2
}

theorem nat_add_4_1 {
    4 + 1 = 5
} by {
}

theorem nat_add_4_2 {
    4 + 2 = 6
} by {
    nat_add_4_1
}

theorem nat_add_4_3 {
    4 + 3 = 7
} by {
    nat_add_4_2
}

theorem nat_add_4_4 {
    4 + 4 = 8
} by {
    nat_add_4_3
}

theorem nat_add_5_1 {
    5 + 1 = 6
} by {
}

theorem nat_add_5_2 {
    5 + 2 = 7
} by {
    nat_add_5_1
}

theorem nat_add_5_3 {
    5 + 3 = 8
} by {
    nat_add_5_2
}

theorem nat_add_5_4 {
    5 + 4 = 9
} by {
    nat_add_5_3
}

theorem nat_add_5_5 {
    5 + 5 = 10
} by {
    nat_add_5_4
}

theorem nat_add_6_1 {
    6 + 1 = 7
} by {
}

theorem nat_add_6_2 {
    6 + 2 = 8
} by {
    nat_add_6_1
}

theorem nat_add_6_3 {
    6 + 3 = 9
} by {
    nat_add_6_2
}

theorem nat_add_6_4 {
    6 + 4 = 10
} by {
    nat_add_6_3
}

theorem nat_add_6_5 {
    6 + 5 = 11
} by {
    nat_add_6_4
}

theorem nat_add_6_6 {
    6 + 6 = 12
} by {
    6 = 5.suc
    6 + 6 = (6 + 5).suc
    nat_add_6_5
    6 + 5 = 11
    12 = 11.suc
}

theorem nat_add_7_1 {
    7 + 1 = 8
} by {
}

theorem nat_add_7_2 {
    7 + 2 = 9
} by {
    nat_add_7_1
}

theorem nat_add_7_3 {
    7 + 3 = 10
} by {
    nat_add_7_2
}

theorem nat_add_7_4 {
    7 + 4 = 11
} by {
    nat_add_7_3
}

theorem nat_add_7_5 {
    7 + 5 = 12
} by {
    5 = 4.suc
    7 + 5 = (7 + 4).suc
    nat_add_7_4
    7 + 4 = 11
    12 = 11.suc
}

theorem nat_add_7_6 {
    7 + 6 = 13
} by {
    6 = 5.suc
    7 + 6 = (7 + 5).suc
    nat_add_7_5
    7 + 5 = 12
    13 = 12.suc
}

theorem nat_add_7_7 {
    7 + 7 = 14
} by {
    7 = 6.suc
    7 + 7 = (7 + 6).suc
    nat_add_7_6
    7 + 6 = 13
    14 = 13.suc
}

theorem nat_add_8_1 {
    8 + 1 = 9
} by {
}

theorem nat_add_8_2 {
    8 + 2 = 10
} by {
    nat_add_8_1
}

theorem nat_add_8_3 {
    8 + 3 = 11
} by {
    nat_add_8_2
}

theorem nat_add_8_4 {
    8 + 4 = 12
} by {
    4 = 3.suc
    8 + 4 = (8 + 3).suc
    nat_add_8_3
    8 + 3 = 11
    12 = 11.suc
}

theorem nat_add_8_5 {
    8 + 5 = 13
} by {
    5 = 4.suc
    8 + 5 = (8 + 4).suc
    nat_add_8_4
    8 + 4 = 12
    13 = 12.suc
}

theorem nat_add_8_6 {
    8 + 6 = 14
} by {
    6 = 5.suc
    8 + 6 = (8 + 5).suc
    nat_add_8_5
    8 + 5 = 13
    14 = 13.suc
}

theorem nat_add_8_7 {
    8 + 7 = 15
} by {
    7 = 6.suc
    8 + 7 = (8 + 6).suc
    nat_add_8_6
    8 + 6 = 14
    15 = 14.suc
}

theorem nat_add_8_8 {
    8 + 8 = 16
} by {
    8 = 7.suc
    8 + 8 = (8 + 7).suc
    nat_add_8_7
    8 + 7 = 15
    16 = 15.suc
}

theorem nat_add_9_1 {
    9 + 1 = 10
} by {
}

theorem nat_add_9_2 {
    9 + 2 = 11
} by {
    nat_add_9_1
}

theorem nat_add_9_3 {
    9 + 3 = 12
} by {
    3 = 2.suc
    9 + 3 = (9 + 2).suc
    nat_add_9_2
    9 + 2 = 11
    12 = 11.suc
}

theorem nat_add_9_4 {
    9 + 4 = 13
} by {
    4 = 3.suc
    9 + 4 = (9 + 3).suc
    nat_add_9_3
    9 + 3 = 12
    13 = 12.suc
}

theorem nat_add_9_5 {
    9 + 5 = 14
} by {
    5 = 4.suc
    9 + 5 = (9 + 4).suc
    nat_add_9_4
    9 + 4 = 13
    14 = 13.suc
}

theorem nat_add_9_6 {
    9 + 6 = 15
} by {
    6 = 5.suc
    9 + 6 = (9 + 5).suc
    nat_add_9_5
    9 + 5 = 14
    15 = 14.suc
}

theorem nat_add_9_7 {
    9 + 7 = 16
} by {
    7 = 6.suc
    9 + 7 = (9 + 6).suc
    nat_add_9_6
    9 + 6 = 15
    16 = 15.suc
}

theorem nat_add_9_8 {
    9 + 8 = 17
} by {
    8 = 7.suc
    9 + 8 = (9 + 7).suc
    nat_add_9_7
    9 + 7 = 16
    17 = 16.suc
}

theorem nat_add_9_9 {
    9 + 9 = 18
} by {
    9 = 8.suc
    9 + 9 = (9 + 8).suc
    nat_add_9_8
    9 + 8 = 17
}

theorem nat_add_10_1 {
    10 + 1 = 11
} by {
}

theorem nat_add_10_2 {
    10 + 2 = 12
} by {
    10 + 2 = (10 + 1).suc
    nat_add_10_1
}

theorem nat_add_10_3 {
    10 + 3 = 13
} by {
    10 + 3 = (10 + 2).suc
    nat_add_10_2
}

theorem nat_add_10_4 {
    10 + 4 = 14
} by {
    10 + 4 = (10 + 3).suc
    nat_add_10_3
}

theorem nat_add_10_5 {
    10 + 5 = 15
} by {
    10 + 5 = (10 + 4).suc
    nat_add_10_4
}

theorem nat_add_12_1 {
    12 + 1 = 13
} by {
    1 = 0.suc
    12 + 1 = (12 + 0).suc
    12 + 0 = 12
    13 = 12.suc
}

theorem nat_add_12_2 {
    12 + 2 = 14
} by {
    2 = 1.suc
    12 + 2 = (12 + 1).suc
    nat_add_12_1
    12 + 1 = 13
    14 = 13.suc
}

theorem nat_add_12_3 {
    12 + 3 = 15
} by {
    3 = 2.suc
    12 + 3 = (12 + 2).suc
    nat_add_12_2
    12 + 2 = 14
    15 = 14.suc
}

theorem nat_add_12_4 {
    12 + 4 = 16
} by {
    4 = 3.suc
    12 + 4 = (12 + 3).suc
    nat_add_12_3
    12 + 3 = 15
    16 = 15.suc
}

theorem nat_add_12_5 {
    12 + 5 = 17
} by {
    5 = 4.suc
    12 + 5 = (12 + 4).suc
    nat_add_12_4
    12 + 4 = 16
    17 = 16.suc
}

theorem nat_add_12_6 {
    12 + 6 = 18
} by {
    6 = 5.suc
    12 + 6 = (12 + 5).suc
    nat_add_12_5
    12 + 5 = 17
    18 = 17.suc
}

theorem nat_add_14_1 {
    14 + 1 = 15
} by {
    1 = 0.suc
    14 + 1 = (14 + 0).suc
    14 + 0 = 14
    15 = 14.suc
}

theorem nat_add_14_2 {
    14 + 2 = 16
} by {
    2 = 1.suc
    14 + 2 = (14 + 1).suc
    nat_add_14_1
    14 + 1 = 15
    16 = 15.suc
}

theorem nat_add_14_3 {
    14 + 3 = 17
} by {
    3 = 2.suc
    14 + 3 = (14 + 2).suc
    nat_add_14_2
    14 + 2 = 16
    17 = 16.suc
}

theorem nat_add_14_4 {
    14 + 4 = 18
} by {
    4 = 3.suc
    14 + 4 = (14 + 3).suc
    nat_add_14_3
    14 + 3 = 17
    18 = 17.suc
}

theorem nat_add_14_5 {
    14 + 5 = 19
} by {
    5 = 4.suc
    14 + 5 = (14 + 4).suc
    nat_add_14_4
    14 + 4 = 18
    19 = 18.suc
}

theorem nat_add_14_6 {
    14 + 6 = 20
} by {
    6 = 5.suc
    14 + 6 = (14 + 5).suc
    nat_add_14_5
    14 + 5 = 19
    20 = 19.suc
}

theorem nat_add_14_7 {
    14 + 7 = 21
} by {
    7 = 6.suc
    14 + 7 = (14 + 6).suc
    nat_add_14_6
    14 + 6 = 20
    21 = 20.suc
}

theorem nat_add_15_1 {
    15 + 1 = 16
} by {
    1 = 0.suc
    15 + 1 = (15 + 0).suc
    15 + 0 = 15
    16 = 15.suc
}

theorem nat_add_15_2 {
    15 + 2 = 17
} by {
    2 = 1.suc
    15 + 2 = (15 + 1).suc
    nat_add_15_1
    15 + 1 = 16
    17 = 16.suc
}

theorem nat_add_15_3 {
    15 + 3 = 18
} by {
    3 = 2.suc
    15 + 3 = (15 + 2).suc
    nat_add_15_2
    15 + 2 = 17
    18 = 17.suc
}

theorem nat_add_15_4 {
    15 + 4 = 19
} by {
    4 = 3.suc
    15 + 4 = (15 + 3).suc
    nat_add_15_3
    15 + 3 = 18
    19 = 18.suc
}

theorem nat_add_15_5 {
    15 + 5 = 20
} by {
    5 = 4.suc
    15 + 5 = (15 + 4).suc
    nat_add_15_4
    15 + 4 = 19
    20 = 19.suc
}

theorem nat_add_16_1 {
    16 + 1 = 17
} by {
    1 = 0.suc
    16 + 1 = (16 + 0).suc
    16 + 0 = 16
    17 = 16.suc
}

theorem nat_add_16_2 {
    16 + 2 = 18
} by {
    2 = 1.suc
    16 + 2 = (16 + 1).suc
    nat_add_16_1
    16 + 1 = 17
    18 = 17.suc
}

theorem nat_add_16_3 {
    16 + 3 = 19
} by {
    3 = 2.suc
    16 + 3 = (16 + 2).suc
    nat_add_16_2
    16 + 2 = 18
    19 = 18.suc
}

theorem nat_add_16_4 {
    16 + 4 = 20
} by {
    4 = 3.suc
    16 + 4 = (16 + 3).suc
    nat_add_16_3
    16 + 3 = 19
    20 = 19.suc
}

theorem nat_add_16_5 {
    16 + 5 = 21
} by {
    5 = 4.suc
    16 + 5 = (16 + 4).suc
    nat_add_16_4
    16 + 4 = 20
    21 = 20.suc
}

theorem nat_add_16_6 {
    16 + 6 = 22
} by {
    6 = 5.suc
    16 + 6 = (16 + 5).suc
    nat_add_16_5
    16 + 5 = 21
    22 = 21.suc
}

theorem nat_add_16_7 {
    16 + 7 = 23
} by {
    7 = 6.suc
    16 + 7 = (16 + 6).suc
    nat_add_16_6
    16 + 6 = 22
    23 = 22.suc
}

theorem nat_add_16_8 {
    16 + 8 = 24
} by {
    8 = 7.suc
    16 + 8 = (16 + 7).suc
    nat_add_16_7
    16 + 7 = 23
    24 = 23.suc
}

theorem nat_add_18_1 {
    18 + 1 = 19
} by {
    1 = 0.suc
    18 + 1 = (18 + 0).suc
    18 + 0 = 18
    19 = 18.suc
}

theorem nat_add_18_2 {
    18 + 2 = 20
} by {
    2 = 1.suc
    18 + 2 = (18 + 1).suc
    nat_add_18_1
    18 + 1 = 19
    20 = 19.suc
}

theorem nat_add_18_3 {
    18 + 3 = 21
} by {
    3 = 2.suc
    18 + 3 = (18 + 2).suc
    nat_add_18_2
    18 + 2 = 20
    21 = 20.suc
}

theorem nat_add_18_4 {
    18 + 4 = 22
} by {
    4 = 3.suc
    18 + 4 = (18 + 3).suc
    nat_add_18_3
    18 + 3 = 21
    22 = 21.suc
}

theorem nat_add_18_5 {
    18 + 5 = 23
} by {
    5 = 4.suc
    18 + 5 = (18 + 4).suc
    nat_add_18_4
    18 + 4 = 22
    23 = 22.suc
}

theorem nat_add_18_6 {
    18 + 6 = 24
} by {
    6 = 5.suc
    18 + 6 = (18 + 5).suc
    nat_add_18_5
    18 + 5 = 23
    24 = 23.suc
}

theorem nat_add_18_7 {
    18 + 7 = 25
} by {
    7 = 6.suc
    18 + 7 = (18 + 6).suc
    nat_add_18_6
    18 + 6 = 24
    25 = 24.suc
}

theorem nat_add_18_8 {
    18 + 8 = 26
} by {
    8 = 7.suc
    18 + 8 = (18 + 7).suc
    nat_add_18_7
    18 + 7 = 25
    26 = 25.suc
}

theorem nat_add_18_9 {
    18 + 9 = 27
} by {
    9 = 8.suc
    18 + 9 = (18 + 8).suc
    nat_add_18_8
    18 + 8 = 26
    27 = 26.suc
}

theorem nat_add_20_1 {
    20 + 1 = 21
} by {
    1 = 0.suc
    20 + 1 = (20 + 0).suc
    20 + 0 = 20
    21 = 20.suc
}

theorem nat_add_20_2 {
    20 + 2 = 22
} by {
    2 = 1.suc
    20 + 2 = (20 + 1).suc
    nat_add_20_1
    20 + 1 = 21
    22 = 21.suc
}

theorem nat_add_20_3 {
    20 + 3 = 23
} by {
    3 = 2.suc
    20 + 3 = (20 + 2).suc
    nat_add_20_2
    20 + 2 = 22
    23 = 22.suc
}

theorem nat_add_20_4 {
    20 + 4 = 24
} by {
    4 = 3.suc
    20 + 4 = (20 + 3).suc
    nat_add_20_3
    20 + 3 = 23
    24 = 23.suc
}

theorem nat_add_20_5 {
    20 + 5 = 25
} by {
    5 = 4.suc
    20 + 5 = (20 + 4).suc
    nat_add_20_4
    20 + 4 = 24
    25 = 24.suc
}

theorem nat_add_21_1 {
    21 + 1 = 22
} by {
    1 = 0.suc
    21 + 1 = (21 + 0).suc
    21 + 0 = 21
    22 = 21.suc
}

theorem nat_add_21_2 {
    21 + 2 = 23
} by {
    2 = 1.suc
    21 + 2 = (21 + 1).suc
    nat_add_21_1
    21 + 1 = 22
    23 = 22.suc
}

theorem nat_add_21_3 {
    21 + 3 = 24
} by {
    3 = 2.suc
    21 + 3 = (21 + 2).suc
    nat_add_21_2
    21 + 2 = 23
    24 = 23.suc
}

theorem nat_add_21_4 {
    21 + 4 = 25
} by {
    4 = 3.suc
    21 + 4 = (21 + 3).suc
    nat_add_21_3
    21 + 3 = 24
    25 = 24.suc
}

theorem nat_add_21_5 {
    21 + 5 = 26
} by {
    5 = 4.suc
    21 + 5 = (21 + 4).suc
    nat_add_21_4
    21 + 4 = 25
    26 = 25.suc
}

theorem nat_add_21_6 {
    21 + 6 = 27
} by {
    6 = 5.suc
    21 + 6 = (21 + 5).suc
    nat_add_21_5
    21 + 5 = 26
    27 = 26.suc
}

theorem nat_add_21_7 {
    21 + 7 = 28
} by {
    7 = 6.suc
    21 + 7 = (21 + 6).suc
    nat_add_21_6
    21 + 6 = 27
    28 = 27.suc
}

theorem nat_add_24_1 {
    24 + 1 = 25
} by {
    1 = 0.suc
    24 + 1 = (24 + 0).suc
    24 + 0 = 24
    25 = 24.suc
}

theorem nat_add_24_2 {
    24 + 2 = 26
} by {
    2 = 1.suc
    24 + 2 = (24 + 1).suc
    nat_add_24_1
    24 + 1 = 25
    26 = 25.suc
}

theorem nat_add_24_3 {
    24 + 3 = 27
} by {
    3 = 2.suc
    24 + 3 = (24 + 2).suc
    nat_add_24_2
    24 + 2 = 26
    27 = 26.suc
}

theorem nat_add_24_4 {
    24 + 4 = 28
} by {
    4 = 3.suc
    24 + 4 = (24 + 3).suc
    nat_add_24_3
    24 + 3 = 27
    28 = 27.suc
}

theorem nat_add_24_5 {
    24 + 5 = 29
} by {
    5 = 4.suc
    24 + 5 = (24 + 4).suc
    nat_add_24_4
    24 + 4 = 28
    29 = 28.suc
}

theorem nat_add_24_6 {
    24 + 6 = 30
} by {
    6 = 5.suc
    24 + 6 = (24 + 5).suc
    nat_add_24_5
    24 + 5 = 29
    30 = 29.suc
}

theorem nat_add_24_7 {
    24 + 7 = 31
} by {
    7 = 6.suc
    24 + 7 = (24 + 6).suc
    nat_add_24_6
    24 + 6 = 30
    31 = 30.suc
}

theorem nat_add_24_8 {
    24 + 8 = 32
} by {
    8 = 7.suc
    24 + 8 = (24 + 7).suc
    nat_add_24_7
    24 + 7 = 31
    32 = 31.suc
}

theorem nat_add_25_1 {
    25 + 1 = 26
} by {
    1 = 0.suc
    25 + 1 = (25 + 0).suc
    25 + 0 = 25
    26 = 25.suc
}

theorem nat_add_25_2 {
    25 + 2 = 27
} by {
    2 = 1.suc
    25 + 2 = (25 + 1).suc
    nat_add_25_1
    25 + 1 = 26
    27 = 26.suc
}

theorem nat_add_25_3 {
    25 + 3 = 28
} by {
    3 = 2.suc
    25 + 3 = (25 + 2).suc
    nat_add_25_2
    25 + 2 = 27
    28 = 27.suc
}

theorem nat_add_25_4 {
    25 + 4 = 29
} by {
    4 = 3.suc
    25 + 4 = (25 + 3).suc
    nat_add_25_3
    25 + 3 = 28
    29 = 28.suc
}

theorem nat_add_25_5 {
    25 + 5 = 30
} by {
    5 = 4.suc
    25 + 5 = (25 + 4).suc
    nat_add_25_4
    25 + 4 = 29
    30 = 29.suc
}

theorem nat_add_27_1 {
    27 + 1 = 28
} by {
    1 = 0.suc
    27 + 1 = (27 + 0).suc
    27 + 0 = 27
    28 = 27.suc
}

theorem nat_add_27_2 {
    27 + 2 = 29
} by {
    2 = 1.suc
    27 + 2 = (27 + 1).suc
    nat_add_27_1
    27 + 1 = 28
    29 = 28.suc
}

theorem nat_add_27_3 {
    27 + 3 = 30
} by {
    3 = 2.suc
    27 + 3 = (27 + 2).suc
    nat_add_27_2
    27 + 2 = 29
    30 = 29.suc
}

theorem nat_add_27_4 {
    27 + 4 = 31
} by {
    4 = 3.suc
    27 + 4 = (27 + 3).suc
    nat_add_27_3
    27 + 3 = 30
    31 = 30.suc
}

theorem nat_add_27_5 {
    27 + 5 = 32
} by {
    5 = 4.suc
    27 + 5 = (27 + 4).suc
    nat_add_27_4
    27 + 4 = 31
    32 = 31.suc
}

theorem nat_add_27_6 {
    27 + 6 = 33
} by {
    6 = 5.suc
    27 + 6 = (27 + 5).suc
    nat_add_27_5
    27 + 5 = 32
    33 = 32.suc
}

theorem nat_add_27_7 {
    27 + 7 = 34
} by {
    7 = 6.suc
    27 + 7 = (27 + 6).suc
    nat_add_27_6
    27 + 6 = 33
    34 = 33.suc
}

theorem nat_add_27_8 {
    27 + 8 = 35
} by {
    8 = 7.suc
    27 + 8 = (27 + 7).suc
    nat_add_27_7
    27 + 7 = 34
    35 = 34.suc
}

theorem nat_add_27_9 {
    27 + 9 = 36
} by {
    9 = 8.suc
    27 + 9 = (27 + 8).suc
    nat_add_27_8
    27 + 8 = 35
    36 = 35.suc
}

theorem nat_add_28_1 {
    28 + 1 = 29
} by {
    1 = 0.suc
    28 + 1 = (28 + 0).suc
    28 + 0 = 28
    29 = 28.suc
}

theorem nat_add_28_2 {
    28 + 2 = 30
} by {
    2 = 1.suc
    28 + 2 = (28 + 1).suc
    nat_add_28_1
    28 + 1 = 29
    30 = 29.suc
}

theorem nat_add_28_3 {
    28 + 3 = 31
} by {
    3 = 2.suc
    28 + 3 = (28 + 2).suc
    nat_add_28_2
    28 + 2 = 30
    31 = 30.suc
}

theorem nat_add_28_4 {
    28 + 4 = 32
} by {
    4 = 3.suc
    28 + 4 = (28 + 3).suc
    nat_add_28_3
    28 + 3 = 31
    32 = 31.suc
}

theorem nat_add_28_5 {
    28 + 5 = 33
} by {
    5 = 4.suc
    28 + 5 = (28 + 4).suc
    nat_add_28_4
    28 + 4 = 32
    33 = 32.suc
}

theorem nat_add_28_6 {
    28 + 6 = 34
} by {
    6 = 5.suc
    28 + 6 = (28 + 5).suc
    nat_add_28_5
    28 + 5 = 33
    34 = 33.suc
}

theorem nat_add_28_7 {
    28 + 7 = 35
} by {
    7 = 6.suc
    28 + 7 = (28 + 6).suc
    nat_add_28_6
    28 + 6 = 34
    35 = 34.suc
}

theorem nat_add_30_1 {
    30 + 1 = 31
} by {
    1 = 0.suc
    30 + 1 = (30 + 0).suc
    30 + 0 = 30
    31 = 30.suc
}

theorem nat_add_30_2 {
    30 + 2 = 32
} by {
    2 = 1.suc
    30 + 2 = (30 + 1).suc
    nat_add_30_1
    30 + 1 = 31
    32 = 31.suc
}

theorem nat_add_30_3 {
    30 + 3 = 33
} by {
    3 = 2.suc
    30 + 3 = (30 + 2).suc
    nat_add_30_2
    30 + 2 = 32
    33 = 32.suc
}

theorem nat_add_30_4 {
    30 + 4 = 34
} by {
    4 = 3.suc
    30 + 4 = (30 + 3).suc
    nat_add_30_3
    30 + 3 = 33
    34 = 33.suc
}

theorem nat_add_30_5 {
    30 + 5 = 35
} by {
    5 = 4.suc
    30 + 5 = (30 + 4).suc
    nat_add_30_4
    30 + 4 = 34
    35 = 34.suc
}

theorem nat_add_30_6 {
    30 + 6 = 36
} by {
    6 = 5.suc
    30 + 6 = (30 + 5).suc
    nat_add_30_5
    30 + 5 = 35
    36 = 35.suc
}

theorem nat_add_32_1 {
    32 + 1 = 33
} by {
    1 = 0.suc
    32 + 1 = (32 + 0).suc
    32 + 0 = 32
    33 = 32.suc
}

theorem nat_add_32_2 {
    32 + 2 = 34
} by {
    2 = 1.suc
    32 + 2 = (32 + 1).suc
    nat_add_32_1
    32 + 1 = 33
    34 = 33.suc
}

theorem nat_add_32_3 {
    32 + 3 = 35
} by {
    3 = 2.suc
    32 + 3 = (32 + 2).suc
    nat_add_32_2
    32 + 2 = 34
    35 = 34.suc
}

theorem nat_add_32_4 {
    32 + 4 = 36
} by {
    4 = 3.suc
    32 + 4 = (32 + 3).suc
    nat_add_32_3
    32 + 3 = 35
    36 = 35.suc
}

theorem nat_add_32_5 {
    32 + 5 = 37
} by {
    5 = 4.suc
    32 + 5 = (32 + 4).suc
    nat_add_32_4
    32 + 4 = 36
    37 = 36.suc
}

theorem nat_add_32_6 {
    32 + 6 = 38
} by {
    6 = 5.suc
    32 + 6 = (32 + 5).suc
    nat_add_32_5
    32 + 5 = 37
    38 = 37.suc
}

theorem nat_add_32_7 {
    32 + 7 = 39
} by {
    7 = 6.suc
    32 + 7 = (32 + 6).suc
    nat_add_32_6
    32 + 6 = 38
    39 = 38.suc
}

theorem nat_add_32_8 {
    32 + 8 = 40
} by {
    8 = 7.suc
    32 + 8 = (32 + 7).suc
    nat_add_32_7
    32 + 7 = 39
    40 = 39.suc
}

theorem nat_add_35_1 {
    35 + 1 = 36
} by {
    1 = 0.suc
    35 + 1 = (35 + 0).suc
    35 + 0 = 35
    36 = 35.suc
}

theorem nat_add_35_2 {
    35 + 2 = 37
} by {
    2 = 1.suc
    35 + 2 = (35 + 1).suc
    nat_add_35_1
    35 + 1 = 36
    37 = 36.suc
}

theorem nat_add_35_3 {
    35 + 3 = 38
} by {
    3 = 2.suc
    35 + 3 = (35 + 2).suc
    nat_add_35_2
    35 + 2 = 37
    38 = 37.suc
}

theorem nat_add_35_4 {
    35 + 4 = 39
} by {
    4 = 3.suc
    35 + 4 = (35 + 3).suc
    nat_add_35_3
    35 + 3 = 38
    39 = 38.suc
}

theorem nat_add_35_5 {
    35 + 5 = 40
} by {
    5 = 4.suc
    35 + 5 = (35 + 4).suc
    nat_add_35_4
    35 + 4 = 39
    40 = 39.suc
}

theorem nat_add_35_6 {
    35 + 6 = 41
} by {
    6 = 5.suc
    35 + 6 = (35 + 5).suc
    nat_add_35_5
    35 + 5 = 40
    41 = 40.suc
}

theorem nat_add_35_7 {
    35 + 7 = 42
} by {
    7 = 6.suc
    35 + 7 = (35 + 6).suc
    nat_add_35_6
    35 + 6 = 41
    42 = 41.suc
}

theorem nat_add_36_1 {
    36 + 1 = 37
} by {
    1 = 0.suc
    36 + 1 = (36 + 0).suc
    36 + 0 = 36
    37 = 36.suc
}

theorem nat_add_36_2 {
    36 + 2 = 38
} by {
    2 = 1.suc
    36 + 2 = (36 + 1).suc
    nat_add_36_1
    36 + 1 = 37
    38 = 37.suc
}

theorem nat_add_36_3 {
    36 + 3 = 39
} by {
    3 = 2.suc
    36 + 3 = (36 + 2).suc
    nat_add_36_2
    36 + 2 = 38
    39 = 38.suc
}

theorem nat_add_36_4 {
    36 + 4 = 40
} by {
    4 = 3.suc
    36 + 4 = (36 + 3).suc
    nat_add_36_3
    36 + 3 = 39
    40 = 39.suc
}

theorem nat_add_36_5 {
    36 + 5 = 41
} by {
    5 = 4.suc
    36 + 5 = (36 + 4).suc
    nat_add_36_4
    36 + 4 = 40
    41 = 40.suc
}

theorem nat_add_36_6 {
    36 + 6 = 42
} by {
    6 = 5.suc
    36 + 6 = (36 + 5).suc
    nat_add_36_5
    36 + 5 = 41
    42 = 41.suc
}

theorem nat_add_36_7 {
    36 + 7 = 43
} by {
    7 = 6.suc
    36 + 7 = (36 + 6).suc
    nat_add_36_6
    36 + 6 = 42
    43 = 42.suc
}

theorem nat_add_36_8 {
    36 + 8 = 44
} by {
    8 = 7.suc
    36 + 8 = (36 + 7).suc
    nat_add_36_7
    36 + 7 = 43
    44 = 43.suc
}

theorem nat_add_36_9 {
    36 + 9 = 45
} by {
    9 = 8.suc
    36 + 9 = (36 + 8).suc
    nat_add_36_8
    36 + 8 = 44
    45 = 44.suc
}

theorem nat_add_40_1 {
    40 + 1 = 41
} by {
    1 = 0.suc
    40 + 1 = (40 + 0).suc
    40 + 0 = 40
    41 = 40.suc
}

theorem nat_add_40_2 {
    40 + 2 = 42
} by {
    2 = 1.suc
    40 + 2 = (40 + 1).suc
    nat_add_40_1
    40 + 1 = 41
    42 = 41.suc
}

theorem nat_add_40_3 {
    40 + 3 = 43
} by {
    3 = 2.suc
    40 + 3 = (40 + 2).suc
    nat_add_40_2
    40 + 2 = 42
    43 = 42.suc
}

theorem nat_add_40_4 {
    40 + 4 = 44
} by {
    4 = 3.suc
    40 + 4 = (40 + 3).suc
    nat_add_40_3
    40 + 3 = 43
    44 = 43.suc
}

theorem nat_add_40_5 {
    40 + 5 = 45
} by {
    5 = 4.suc
    40 + 5 = (40 + 4).suc
    nat_add_40_4
    40 + 4 = 44
    45 = 44.suc
}

theorem nat_add_40_6 {
    40 + 6 = 46
} by {
    6 = 5.suc
    40 + 6 = (40 + 5).suc
    nat_add_40_5
    40 + 5 = 45
    46 = 45.suc
}

theorem nat_add_40_7 {
    40 + 7 = 47
} by {
    7 = 6.suc
    40 + 7 = (40 + 6).suc
    nat_add_40_6
    40 + 6 = 46
    47 = 46.suc
}

theorem nat_add_40_8 {
    40 + 8 = 48
} by {
    8 = 7.suc
    40 + 8 = (40 + 7).suc
    nat_add_40_7
    40 + 7 = 47
    48 = 47.suc
}

theorem nat_add_42_1 {
    42 + 1 = 43
} by {
    1 = 0.suc
    42 + 1 = (42 + 0).suc
    42 + 0 = 42
    43 = 42.suc
}

theorem nat_add_42_2 {
    42 + 2 = 44
} by {
    2 = 1.suc
    42 + 2 = (42 + 1).suc
    nat_add_42_1
    42 + 1 = 43
    44 = 43.suc
}

theorem nat_add_42_3 {
    42 + 3 = 45
} by {
    3 = 2.suc
    42 + 3 = (42 + 2).suc
    nat_add_42_2
    42 + 2 = 44
    45 = 44.suc
}

theorem nat_add_42_4 {
    42 + 4 = 46
} by {
    4 = 3.suc
    42 + 4 = (42 + 3).suc
    nat_add_42_3
    42 + 3 = 45
    46 = 45.suc
}

theorem nat_add_42_5 {
    42 + 5 = 47
} by {
    5 = 4.suc
    42 + 5 = (42 + 4).suc
    nat_add_42_4
    42 + 4 = 46
    47 = 46.suc
}

theorem nat_add_42_6 {
    42 + 6 = 48
} by {
    6 = 5.suc
    42 + 6 = (42 + 5).suc
    nat_add_42_5
    42 + 5 = 47
    48 = 47.suc
}

theorem nat_add_42_7 {
    42 + 7 = 49
} by {
    7 = 6.suc
    42 + 7 = (42 + 6).suc
    nat_add_42_6
    42 + 6 = 48
    49 = 48.suc
}

theorem nat_add_45_1 {
    45 + 1 = 46
} by {
    1 = 0.suc
    45 + 1 = (45 + 0).suc
    45 + 0 = 45
    46 = 45.suc
}

theorem nat_add_45_2 {
    45 + 2 = 47
} by {
    2 = 1.suc
    45 + 2 = (45 + 1).suc
    nat_add_45_1
    45 + 1 = 46
    47 = 46.suc
}

theorem nat_add_45_3 {
    45 + 3 = 48
} by {
    3 = 2.suc
    45 + 3 = (45 + 2).suc
    nat_add_45_2
    45 + 2 = 47
    48 = 47.suc
}

theorem nat_add_45_4 {
    45 + 4 = 49
} by {
    4 = 3.suc
    45 + 4 = (45 + 3).suc
    nat_add_45_3
    45 + 3 = 48
    49 = 48.suc
}

theorem nat_add_45_5 {
    45 + 5 = 50
} by {
    5 = 4.suc
    45 + 5 = (45 + 4).suc
    nat_add_45_4
    45 + 4 = 49
    50 = 49.suc
}

theorem nat_add_45_6 {
    45 + 6 = 51
} by {
    6 = 5.suc
    45 + 6 = (45 + 5).suc
    nat_add_45_5
    45 + 5 = 50
    51 = 50.suc
}

theorem nat_add_45_7 {
    45 + 7 = 52
} by {
    7 = 6.suc
    45 + 7 = (45 + 6).suc
    nat_add_45_6
    45 + 6 = 51
    52 = 51.suc
}

theorem nat_add_45_8 {
    45 + 8 = 53
} by {
    8 = 7.suc
    45 + 8 = (45 + 7).suc
    nat_add_45_7
    45 + 7 = 52
    53 = 52.suc
}

theorem nat_add_45_9 {
    45 + 9 = 54
} by {
    9 = 8.suc
    45 + 9 = (45 + 8).suc
    nat_add_45_8
    45 + 8 = 53
    54 = 53.suc
}

theorem nat_add_48_1 {
    48 + 1 = 49
} by {
    1 = 0.suc
    48 + 1 = (48 + 0).suc
    48 + 0 = 48
    49 = 48.suc
}

theorem nat_add_48_2 {
    48 + 2 = 50
} by {
    2 = 1.suc
    48 + 2 = (48 + 1).suc
    nat_add_48_1
    48 + 1 = 49
    50 = 49.suc
}

theorem nat_add_48_3 {
    48 + 3 = 51
} by {
    3 = 2.suc
    48 + 3 = (48 + 2).suc
    nat_add_48_2
    48 + 2 = 50
    51 = 50.suc
}

theorem nat_add_48_4 {
    48 + 4 = 52
} by {
    4 = 3.suc
    48 + 4 = (48 + 3).suc
    nat_add_48_3
    48 + 3 = 51
    52 = 51.suc
}

theorem nat_add_48_5 {
    48 + 5 = 53
} by {
    5 = 4.suc
    48 + 5 = (48 + 4).suc
    nat_add_48_4
    48 + 4 = 52
    53 = 52.suc
}

theorem nat_add_48_6 {
    48 + 6 = 54
} by {
    6 = 5.suc
    48 + 6 = (48 + 5).suc
    nat_add_48_5
    48 + 5 = 53
    54 = 53.suc
}

theorem nat_add_48_7 {
    48 + 7 = 55
} by {
    7 = 6.suc
    48 + 7 = (48 + 6).suc
    nat_add_48_6
    48 + 6 = 54
    55 = 54.suc
}

theorem nat_add_48_8 {
    48 + 8 = 56
} by {
    8 = 7.suc
    48 + 8 = (48 + 7).suc
    nat_add_48_7
    48 + 7 = 55
    56 = 55.suc
}

theorem nat_add_49_1 {
    49 + 1 = 50
} by {
    1 = 0.suc
    49 + 1 = (49 + 0).suc
    49 + 0 = 49
    50 = 49.suc
}

theorem nat_add_49_2 {
    49 + 2 = 51
} by {
    2 = 1.suc
    49 + 2 = (49 + 1).suc
    nat_add_49_1
    49 + 1 = 50
    51 = 50.suc
}

theorem nat_add_49_3 {
    49 + 3 = 52
} by {
    3 = 2.suc
    49 + 3 = (49 + 2).suc
    nat_add_49_2
    49 + 2 = 51
    52 = 51.suc
}

theorem nat_add_49_4 {
    49 + 4 = 53
} by {
    4 = 3.suc
    49 + 4 = (49 + 3).suc
    nat_add_49_3
    49 + 3 = 52
    53 = 52.suc
}

theorem nat_add_49_5 {
    49 + 5 = 54
} by {
    5 = 4.suc
    49 + 5 = (49 + 4).suc
    nat_add_49_4
    49 + 4 = 53
    54 = 53.suc
}

theorem nat_add_49_6 {
    49 + 6 = 55
} by {
    6 = 5.suc
    49 + 6 = (49 + 5).suc
    nat_add_49_5
    49 + 5 = 54
    55 = 54.suc
}

theorem nat_add_49_7 {
    49 + 7 = 56
} by {
    7 = 6.suc
    49 + 7 = (49 + 6).suc
    nat_add_49_6
    49 + 6 = 55
    56 = 55.suc
}

theorem nat_add_54_1 {
    54 + 1 = 55
} by {
    1 = 0.suc
    54 + 1 = (54 + 0).suc
    54 + 0 = 54
    55 = 54.suc
}

theorem nat_add_54_2 {
    54 + 2 = 56
} by {
    2 = 1.suc
    54 + 2 = (54 + 1).suc
    nat_add_54_1
    54 + 1 = 55
    56 = 55.suc
}

theorem nat_add_54_3 {
    54 + 3 = 57
} by {
    3 = 2.suc
    54 + 3 = (54 + 2).suc
    nat_add_54_2
    54 + 2 = 56
    57 = 56.suc
}

theorem nat_add_54_4 {
    54 + 4 = 58
} by {
    4 = 3.suc
    54 + 4 = (54 + 3).suc
    nat_add_54_3
    54 + 3 = 57
    58 = 57.suc
}

theorem nat_add_54_5 {
    54 + 5 = 59
} by {
    5 = 4.suc
    54 + 5 = (54 + 4).suc
    nat_add_54_4
    54 + 4 = 58
    59 = 58.suc
}

theorem nat_add_54_6 {
    54 + 6 = 60
} by {
    6 = 5.suc
    54 + 6 = (54 + 5).suc
    nat_add_54_5
    54 + 5 = 59
    60 = 59.suc
}

theorem nat_add_54_7 {
    54 + 7 = 61
} by {
    7 = 6.suc
    54 + 7 = (54 + 6).suc
    nat_add_54_6
    54 + 6 = 60
    61 = 60.suc
}

theorem nat_add_54_8 {
    54 + 8 = 62
} by {
    8 = 7.suc
    54 + 8 = (54 + 7).suc
    nat_add_54_7
    54 + 7 = 61
    62 = 61.suc
}

theorem nat_add_54_9 {
    54 + 9 = 63
} by {
    9 = 8.suc
    54 + 9 = (54 + 8).suc
    nat_add_54_8
    54 + 8 = 62
    63 = 62.suc
}

theorem nat_add_56_1 {
    56 + 1 = 57
} by {
    1 = 0.suc
    56 + 1 = (56 + 0).suc
    56 + 0 = 56
    57 = 56.suc
}

theorem nat_add_56_2 {
    56 + 2 = 58
} by {
    2 = 1.suc
    56 + 2 = (56 + 1).suc
    nat_add_56_1
    56 + 1 = 57
    58 = 57.suc
}

theorem nat_add_56_3 {
    56 + 3 = 59
} by {
    3 = 2.suc
    56 + 3 = (56 + 2).suc
    nat_add_56_2
    56 + 2 = 58
    59 = 58.suc
}

theorem nat_add_56_4 {
    56 + 4 = 60
} by {
    4 = 3.suc
    56 + 4 = (56 + 3).suc
    nat_add_56_3
    56 + 3 = 59
    60 = 59.suc
}

theorem nat_add_56_5 {
    56 + 5 = 61
} by {
    5 = 4.suc
    56 + 5 = (56 + 4).suc
    nat_add_56_4
    56 + 4 = 60
    61 = 60.suc
}

theorem nat_add_56_6 {
    56 + 6 = 62
} by {
    6 = 5.suc
    56 + 6 = (56 + 5).suc
    nat_add_56_5
    56 + 5 = 61
    62 = 61.suc
}

theorem nat_add_56_7 {
    56 + 7 = 63
} by {
    7 = 6.suc
    56 + 7 = (56 + 6).suc
    nat_add_56_6
    56 + 6 = 62
    63 = 62.suc
}

theorem nat_add_56_8 {
    56 + 8 = 64
} by {
    8 = 7.suc
    56 + 8 = (56 + 7).suc
    nat_add_56_7
    56 + 7 = 63
    64 = 63.suc
}

theorem nat_add_63_1 {
    63 + 1 = 64
} by {
    1 = 0.suc
    63 + 1 = (63 + 0).suc
    63 + 0 = 63
    64 = 63.suc
}

theorem nat_add_63_2 {
    63 + 2 = 65
} by {
    2 = 1.suc
    63 + 2 = (63 + 1).suc
    nat_add_63_1
    63 + 1 = 64
    65 = 64.suc
}

theorem nat_add_63_3 {
    63 + 3 = 66
} by {
    3 = 2.suc
    63 + 3 = (63 + 2).suc
    nat_add_63_2
    63 + 2 = 65
    66 = 65.suc
}

theorem nat_add_63_4 {
    63 + 4 = 67
} by {
    4 = 3.suc
    63 + 4 = (63 + 3).suc
    nat_add_63_3
    63 + 3 = 66
    67 = 66.suc
}

theorem nat_add_63_5 {
    63 + 5 = 68
} by {
    5 = 4.suc
    63 + 5 = (63 + 4).suc
    nat_add_63_4
    63 + 4 = 67
    68 = 67.suc
}

theorem nat_add_63_6 {
    63 + 6 = 69
} by {
    6 = 5.suc
    63 + 6 = (63 + 5).suc
    nat_add_63_5
    63 + 5 = 68
    69 = 68.suc
}

theorem nat_add_63_7 {
    63 + 7 = 70
} by {
    7 = 6.suc
    63 + 7 = (63 + 6).suc
    nat_add_63_6
    63 + 6 = 69
    70 = 69.suc
}

theorem nat_add_63_8 {
    63 + 8 = 71
} by {
    8 = 7.suc
    63 + 8 = (63 + 7).suc
    nat_add_63_7
    63 + 7 = 70
    71 = 70.suc
}

theorem nat_add_63_9 {
    63 + 9 = 72
} by {
    9 = 8.suc
    63 + 9 = (63 + 8).suc
    nat_add_63_8
    63 + 8 = 71
    72 = 71.suc
}

theorem nat_add_64_1 {
    64 + 1 = 65
} by {
    1 = 0.suc
    64 + 1 = (64 + 0).suc
    64 + 0 = 64
    65 = 64.suc
}

theorem nat_add_64_2 {
    64 + 2 = 66
} by {
    2 = 1.suc
    64 + 2 = (64 + 1).suc
    nat_add_64_1
    64 + 1 = 65
    66 = 65.suc
}

theorem nat_add_64_3 {
    64 + 3 = 67
} by {
    3 = 2.suc
    64 + 3 = (64 + 2).suc
    nat_add_64_2
    64 + 2 = 66
    67 = 66.suc
}

theorem nat_add_64_4 {
    64 + 4 = 68
} by {
    4 = 3.suc
    64 + 4 = (64 + 3).suc
    nat_add_64_3
    64 + 3 = 67
    68 = 67.suc
}

theorem nat_add_64_5 {
    64 + 5 = 69
} by {
    5 = 4.suc
    64 + 5 = (64 + 4).suc
    nat_add_64_4
    64 + 4 = 68
    69 = 68.suc
}

theorem nat_add_64_6 {
    64 + 6 = 70
} by {
    6 = 5.suc
    64 + 6 = (64 + 5).suc
    nat_add_64_5
    64 + 5 = 69
    70 = 69.suc
}

theorem nat_add_64_7 {
    64 + 7 = 71
} by {
    7 = 6.suc
    64 + 7 = (64 + 6).suc
    nat_add_64_6
    64 + 6 = 70
    71 = 70.suc
}

theorem nat_add_64_8 {
    64 + 8 = 72
} by {
    8 = 7.suc
    64 + 8 = (64 + 7).suc
    nat_add_64_7
    64 + 7 = 71
    72 = 71.suc
}

theorem nat_add_72_1 {
    72 + 1 = 73
} by {
    1 = 0.suc
    72 + 1 = (72 + 0).suc
    72 + 0 = 72
    73 = 72.suc
}

theorem nat_add_72_2 {
    72 + 2 = 74
} by {
    2 = 1.suc
    72 + 2 = (72 + 1).suc
    nat_add_72_1
    72 + 1 = 73
    74 = 73.suc
}

theorem nat_add_72_3 {
    72 + 3 = 75
} by {
    3 = 2.suc
    72 + 3 = (72 + 2).suc
    nat_add_72_2
    72 + 2 = 74
    75 = 74.suc
}

theorem nat_add_72_4 {
    72 + 4 = 76
} by {
    4 = 3.suc
    72 + 4 = (72 + 3).suc
    nat_add_72_3
    72 + 3 = 75
    76 = 75.suc
}

theorem nat_add_72_5 {
    72 + 5 = 77
} by {
    5 = 4.suc
    72 + 5 = (72 + 4).suc
    nat_add_72_4
    72 + 4 = 76
    77 = 76.suc
}

theorem nat_add_72_6 {
    72 + 6 = 78
} by {
    6 = 5.suc
    72 + 6 = (72 + 5).suc
    nat_add_72_5
    72 + 5 = 77
    78 = 77.suc
}

theorem nat_add_72_7 {
    72 + 7 = 79
} by {
    7 = 6.suc
    72 + 7 = (72 + 6).suc
    nat_add_72_6
    72 + 6 = 78
    79 = 78.suc
}

theorem nat_add_72_8 {
    72 + 8 = 80
} by {
    8 = 7.suc
    72 + 8 = (72 + 7).suc
    nat_add_72_7
    72 + 7 = 79
    80 = 79.suc
}

theorem nat_add_72_9 {
    72 + 9 = 81
} by {
    9 = 8.suc
    72 + 9 = (72 + 8).suc
    nat_add_72_8
    72 + 8 = 80
    81 = 80.suc
}

/// One-digit multiplication facts for natural numerals from 1 through 9.
theorem nat_mul_1_1 {
    1 * 1 = 1
} by {
    nat_add_0_1
}

theorem nat_mul_1_2 {
    1 * 2 = 2
} by {
    nat_mul_1_1
    nat_add_1_1
}

theorem nat_mul_1_3 {
    1 * 3 = 3
} by {
    nat_mul_1_2
    nat_add_2_1
}

theorem nat_mul_1_4 {
    1 * 4 = 4
} by {
    nat_mul_1_3
    nat_add_3_1
}

theorem nat_mul_1_5 {
    1 * 5 = 5
} by {
    nat_mul_1_4
    nat_add_4_1
}

theorem nat_mul_1_6 {
    1 * 6 = 6
} by {
    nat_mul_1_5
    nat_add_5_1
}

theorem nat_mul_1_7 {
    1 * 7 = 7
} by {
    nat_mul_1_6
    nat_add_6_1
}

theorem nat_mul_1_8 {
    1 * 8 = 8
} by {
    nat_mul_1_7
    nat_add_7_1
}

theorem nat_mul_1_9 {
    1 * 9 = 9
} by {
    nat_mul_1_8
    nat_add_8_1
}

theorem nat_mul_2_1 {
    2 * 1 = 2
} by {
    nat_add_0_2
}

theorem nat_mul_2_2 {
    2 * 2 = 4
} by {
    nat_mul_2_1
    nat_add_2_2
}

theorem nat_mul_2_3 {
    2 * 3 = 6
} by {
    nat_mul_2_2
    nat_add_4_2
}

theorem nat_mul_2_4 {
    2 * 4 = 8
} by {
    nat_mul_2_3
    nat_add_6_2
}

theorem nat_mul_2_5 {
    2 * 5 = 10
} by {
    nat_mul_2_4
    nat_add_8_2
}

theorem nat_mul_2_6 {
    2 * 6 = 12
} by {
    nat_mul_2_5
    nat_add_10_2
}

theorem nat_mul_2_7 {
    2 * 7 = 14
} by {
    nat_mul_2_6
    nat_add_12_2
}

theorem nat_mul_2_8 {
    2 * 8 = 16
} by {
    nat_mul_2_7
    nat_add_14_2
}

theorem nat_mul_2_9 {
    2 * 9 = 18
} by {
    nat_mul_2_8
    nat_add_16_2
}

theorem nat_mul_3_1 {
    3 * 1 = 3
} by {
    nat_add_0_3
}

theorem nat_mul_3_2 {
    3 * 2 = 6
} by {
    nat_mul_3_1
    nat_add_3_3
}

theorem nat_mul_3_3 {
    3 * 3 = 9
} by {
    nat_mul_3_2
    nat_add_6_3
}

theorem nat_mul_3_4 {
    3 * 4 = 12
} by {
    nat_mul_3_3
    nat_add_9_3
}

theorem nat_mul_3_5 {
    3 * 5 = 15
} by {
    3 * 5 = 3 * 4 + 3
    nat_mul_3_4
    nat_add_12_3
}

theorem nat_mul_3_6 {
    3 * 6 = 18
} by {
    3 * 6 = 3 * 5 + 3
    nat_mul_3_5
    nat_add_15_3
}

theorem nat_mul_3_7 {
    3 * 7 = 21
} by {
    3 * 7 = 3 * 6 + 3
    nat_mul_3_6
    nat_add_18_3
}

theorem nat_mul_3_8 {
    3 * 8 = 24
} by {
    nat_mul_3_7
    nat_add_21_3
}

theorem nat_mul_3_9 {
    3 * 9 = 27
} by {
    nat_mul_3_8
    nat_add_24_3
}

theorem nat_mul_4_1 {
    4 * 1 = 4
} by {
    nat_add_0_4
}

theorem nat_mul_4_2 {
    4 * 2 = 8
} by {
    nat_mul_4_1
    nat_add_4_4
}

theorem nat_mul_4_3 {
    4 * 3 = 12
} by {
    nat_mul_4_2
    nat_add_8_4
}

theorem nat_mul_4_4 {
    4 * 4 = 16
} by {
    4 * 4 = 4 * 3 + 4
    nat_mul_4_3
    nat_add_12_4
}

theorem nat_mul_4_5 {
    4 * 5 = 20
} by {
    nat_mul_4_4
    nat_add_16_4
}

theorem nat_mul_4_6 {
    4 * 6 = 24
} by {
    nat_mul_4_5
    nat_add_20_4
}

theorem nat_mul_4_7 {
    4 * 7 = 28
} by {
    nat_mul_4_6
    nat_add_24_4
}

theorem nat_mul_4_8 {
    4 * 8 = 32
} by {
    nat_mul_4_7
    nat_add_28_4
}

theorem nat_mul_4_9 {
    4 * 9 = 36
} by {
    nat_mul_4_8
    nat_add_32_4
}

theorem nat_mul_5_1 {
    5 * 1 = 5
} by {
    nat_add_0_5
}

theorem nat_mul_5_2 {
    5 * 2 = 10
} by {
    nat_mul_5_1
    nat_add_5_5
}

theorem nat_mul_5_3 {
    5 * 3 = 15
} by {
    nat_mul_5_2
    nat_add_10_5
}

theorem nat_mul_5_4 {
    5 * 4 = 20
} by {
    nat_mul_5_3
    nat_add_15_5
}

theorem nat_mul_5_5 {
    5 * 5 = 25
} by {
    nat_mul_5_4
    nat_add_20_5
}

theorem nat_mul_5_6 {
    5 * 6 = 30
} by {
    nat_mul_5_5
    nat_add_25_5
}

theorem nat_mul_5_7 {
    5 * 7 = 35
} by {
    nat_mul_5_6
    nat_add_30_5
}

theorem nat_mul_5_8 {
    5 * 8 = 40
} by {
    nat_mul_5_7
    nat_add_35_5
}

theorem nat_mul_5_9 {
    5 * 9 = 45
} by {
    nat_mul_5_8
    nat_add_40_5
}

theorem nat_mul_6_1 {
    6 * 1 = 6
} by {
    nat_add_0_6
}

theorem nat_mul_6_2 {
    6 * 2 = 12
} by {
    nat_mul_6_1
    nat_add_6_6
}

theorem nat_mul_6_3 {
    6 * 3 = 18
} by {
    nat_mul_6_2
    nat_add_12_6
}

theorem nat_mul_6_4 {
    6 * 4 = 24
} by {
    nat_mul_6_3
    nat_add_18_6
}

theorem nat_mul_6_5 {
    6 * 5 = 30
} by {
    nat_mul_6_4
    nat_add_24_6
}

theorem nat_mul_6_6 {
    6 * 6 = 36
} by {
    nat_mul_6_5
    nat_add_30_6
}

theorem nat_mul_6_7 {
    6 * 7 = 42
} by {
    nat_mul_6_6
    nat_add_36_6
}

theorem nat_mul_6_8 {
    6 * 8 = 48
} by {
    nat_mul_6_7
    nat_add_42_6
}

theorem nat_mul_6_9 {
    6 * 9 = 54
} by {
    nat_mul_6_8
    nat_add_48_6
}

theorem nat_mul_7_1 {
    7 * 1 = 7
} by {
    nat_add_0_7
}

theorem nat_mul_7_2 {
    7 * 2 = 14
} by {
    nat_mul_7_1
    nat_add_7_7
}

theorem nat_mul_7_3 {
    7 * 3 = 21
} by {
    nat_mul_7_2
    nat_add_14_7
}

theorem nat_mul_7_4 {
    7 * 4 = 28
} by {
    nat_mul_7_3
    nat_add_21_7
}

theorem nat_mul_7_5 {
    7 * 5 = 35
} by {
    nat_mul_7_4
    nat_add_28_7
}

theorem nat_mul_7_6 {
    7 * 6 = 42
} by {
    nat_mul_7_5
    nat_add_35_7
}

theorem nat_mul_7_7 {
    7 * 7 = 49
} by {
    7 * 7 = 7 * 6 + 7
    nat_mul_7_6
    nat_add_42_7
}

theorem nat_mul_7_8 {
    7 * 8 = 56
} by {
    nat_mul_7_7
    nat_add_49_7
}

theorem nat_mul_7_9 {
    7 * 9 = 63
} by {
    nat_mul_7_8
    nat_add_56_7
}

theorem nat_mul_8_1 {
    8 * 1 = 8
} by {
    nat_add_0_8
}

theorem nat_mul_8_2 {
    8 * 2 = 16
} by {
    nat_mul_8_1
    nat_add_8_8
}

theorem nat_mul_8_3 {
    8 * 3 = 24
} by {
    nat_mul_8_2
    nat_add_16_8
}

theorem nat_mul_8_4 {
    8 * 4 = 32
} by {
    nat_mul_8_3
    nat_add_24_8
}

theorem nat_mul_8_5 {
    8 * 5 = 40
} by {
    nat_mul_8_4
    nat_add_32_8
}

theorem nat_mul_8_6 {
    8 * 6 = 48
} by {
    nat_mul_8_5
    nat_add_40_8
}

theorem nat_mul_8_7 {
    8 * 7 = 56
} by {
    nat_mul_8_6
    nat_add_48_8
}

theorem nat_mul_8_8 {
    8 * 8 = 64
} by {
    8 * 8 = 8 * 7 + 8
    nat_mul_8_7
    nat_add_56_8
}

theorem nat_mul_8_9 {
    8 * 9 = 72
} by {
    nat_mul_8_8
    nat_add_64_8
}

theorem nat_mul_9_1 {
    9 * 1 = 9
} by {
    nat_add_0_9
}

theorem nat_mul_9_2 {
    9 * 2 = 18
} by {
    nat_mul_9_1
    nat_add_9_9
}

theorem nat_mul_9_3 {
    9 * 3 = 27
} by {
    nat_mul_9_2
    nat_add_18_9
}

theorem nat_mul_9_4 {
    9 * 4 = 36
} by {
    nat_mul_9_3
    nat_add_27_9
}

theorem nat_mul_9_5 {
    9 * 5 = 45
} by {
    nat_mul_9_4
    nat_add_36_9
}

theorem nat_mul_9_6 {
    9 * 6 = 54
} by {
    nat_mul_9_5
    nat_add_45_9
}

theorem nat_mul_9_7 {
    9 * 7 = 63
} by {
    nat_mul_9_6
    nat_add_54_9
}

theorem nat_mul_9_8 {
    9 * 8 = 72
} by {
    nat_mul_9_7
    nat_add_63_9
}

theorem nat_mul_9_9 {
    9 * 9 = 81
} by {
    nat_mul_9_8
    nat_add_72_9
}
