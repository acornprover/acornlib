from nat.gcd import Nat, gcd_divides_left, gcd_comm, gcd_nonzero_left
numerals Nat

/// The least common multiple of two natural numbers, characterised by
/// `a.gcd(b) * lcm(a, b) = a * b`. When both inputs are zero the gcd is also
/// zero and the equation collapses; we pick `lcm(0, 0) = 0` by convention.
let nat_lcm(a: Nat, b: Nat) -> l: Nat satisfy {
    if a.gcd(b) = Nat.0 {
        l = Nat.0
    } else {
        a.gcd(b) * l = a * b
    }
} by {
    if a.gcd(b) = Nat.0 {
        let x: Nat satisfy { x = Nat.0 }
    } else {
        gcd_divides_left(a, b)
        let q: Nat satisfy { a.gcd(b) * q = a }
        a.gcd(b) * (q * b) = a * b
        let x: Nat satisfy { a.gcd(b) * x = a * b }
    }
}

attributes Nat {
    /// The least common multiple of this number and b.
    let lcm = nat_lcm
}

/// The defining equation for lcm: gcd times lcm equals the product.
theorem gcd_mul_lcm(a: Nat, b: Nat) {
    a.gcd(b) * a.lcm(b) = a * b
} by {
    if a.gcd(b) = Nat.0 {
        gcd_nonzero_left(a, b)
        gcd_comm(a, b)
        gcd_nonzero_left(b, a)
        Nat.0 * a.lcm(b) = Nat.0
    } else {
    }
}
