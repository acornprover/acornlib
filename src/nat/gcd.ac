from nat.nat_base import Nat, strong_induction, true_below, mul_cancel_left
from pair import Pair
numerals Nat

/// Performs one step of the Euclidean algorithm for computing GCD.
/// Takes the pair (a, b) and replaces it with (b, a mod b).
define gcd_step(p: Pair[Nat, Nat]) -> Pair[Nat, Nat] {
    if p.second = 0 {
        p
    } else {
        Pair.new(p.second, p.first.mod(p.second))
    }
}

/// Performs n steps of the Euclidean GCD algorithm.
define gcd_step_n(p: Pair[Nat, Nat], n: Nat) -> Pair[Nat, Nat] {
    match n {
        Nat.zero {
            p
        }
        Nat.suc(pred) {
            gcd_step(gcd_step_n(p, pred))
        }
    }
}

/// The Euclidean algorithm step viewed as a relation on pairs of natural numbers.
define gcd_step_relation(p: Pair[Nat, Nat], q: Pair[Nat, Nat]) -> Bool {
    gcd_step(p) = q
}

/// True if a predicate is false for all values below n.
define false_below(f: Nat -> Bool, n: Nat) -> Bool { forall(x: Nat) { x < n implies not f(x) } }

/// A predicate false below `n` is false at each smaller number.
theorem false_below_apply(f: Nat -> Bool, n: Nat, x: Nat) {
    false_below(f, n) and x < n implies not f(x)
} by {
    if false_below(f, n) and x < n {
        false_below(f, n) = forall(y: Nat) {
            y < n implies not f(y)
        }
        not f(x)
    }
}

theorem all_false_below(f: Nat -> Bool) {
    forall(x: Nat) { false_below(f, x) } implies forall(x: Nat) { not f(x) }
}

/// True if m is the smallest natural number for which the predicate f is true.
define is_min(f: Nat -> Bool, m: Nat) -> Bool {
    f(m) and false_below(f, m)
}

/// A minimum satisfies its predicate.
theorem is_min_apply(f: Nat -> Bool, m: Nat) {
    is_min(f, m) implies f(m)
} by {
    if is_min(f, m) {
        f(m)
    }
}

/// A minimum has no smaller witness.
theorem is_min_false_below(f: Nat -> Bool, m: Nat) {
    is_min(f, m) implies false_below(f, m)
} by {
    if is_min(f, m) {
        false_below(f, m)
    }
}

// If f is true anywhere on the naturals, it has a min.
theorem has_min(f: Nat -> Bool, n: Nat) {
    f(n) implies exists(m: Nat) { is_min(f, m) }
} by {
    define g(x: Nat) -> Bool { has_min(f, x) }
    forall(k: Nat) {
        if true_below(g, k) {
            if f(k) {
                if false_below(f, k) {
                    exists(m: Nat) { is_min(f, m) }
                    has_min(f, k)
                } else {
                    let m: Nat satisfy { m < k and f(m) }
                    has_min(f, m)
                    has_min(f, k)
                }
            } else {
                has_min(f, k)
            }
        }
    }
    forall(k: Nat) {
        true_below(g, k) implies g(k)
    }
    forall(k: Nat) {
        g(k)
    }
}

// A decreasing_to_zero function strictly decreases until it hits zero.
define decreasing_to_zero(f: Nat -> Nat) -> Bool {
    forall(x: Nat) {
        f(x) = 0 or f(x.suc) < f(x)
    }
}

theorem no_infinite_decreasing(f: Nat -> Nat) {
    exists(x: Nat) { f(x) <= f(x.suc) }
} by {
    if not no_infinite_decreasing(f) {
        let h = function(x: Nat) {
            x + f(x) <= f(0)
        }
        h(0)
        forall(x: Nat) {
            if h(x) {
                f(x.suc) < f(x)
                x + f(x.suc) < x + f(x)
                x.suc + f(x.suc) = (x + f(x.suc)).suc
                (x + f(x.suc)).suc <= x + f(x)
                x.suc + f(x.suc) <= x + f(x)
                h(x.suc)
            }
        }
        forall(x: Nat) {
            h(x)
        }
        h(f(0).suc)
        f(0).suc + f(f(0).suc) <= f(0)
        f(0).suc <= f(0)
        false
    }
}

theorem dtz_terminates(f: Nat -> Nat) {
    decreasing_to_zero(f) implies exists(n: Nat) { f(n) = 0 }
}

theorem gcd_terminates(p: Pair[Nat, Nat]) {
    exists(n: Nat) { gcd_step_n(p, n).second = 0 }
} by {
    let f = function(x: Nat) { gcd_step_n(p, x).second }
    forall(x: Nat) {
        if f(x) != 0 {
            let q = gcd_step_n(p, x)
            Pair.new(q.second, q.first.mod(q.second)) = gcd_step(q)
            Pair.new(q.second, q.first.mod(q.second)).second = q.first.mod(q.second)
            q.first.mod(f(x)) < f(x)
            f(x.suc) < f(x)
        }
    }
    decreasing_to_zero(f)
}

define gcd_termination(p: Pair[Nat, Nat]) -> (Nat -> Bool) {
    function(n: Nat) {
        gcd_step_n(p, n).second = 0
    }
}

let num_gcd_steps(p: Pair[Nat, Nat]) -> n: Nat satisfy {
    gcd_step_n(p, n).second = 0
}

theorem num_gcd_steps_terminates(p: Pair[Nat, Nat]) {
    gcd_step_n(p, num_gcd_steps(p)).second = 0
} by {
    let f = gcd_termination(p)
}

define gcd_pair(p: Pair[Nat, Nat]) -> Nat { gcd_step_n(p, num_gcd_steps(p)).first }

attributes Nat {
    /// The greatest common divisor of this number and b.
    define gcd(self, b: Nat) -> Nat { gcd_pair(Pair.new(self, b)) }
}

define divides_both(a: Nat, p: Pair[Nat, Nat]) -> Bool {
    a.divides(p.first) and a.divides(p.second)
}

/// The relation that records preservation of a common divisor along Euclidean states.
define preserves_divides_both(d: Nat, p: Pair[Nat, Nat], q: Pair[Nat, Nat]) -> Bool {
    divides_both(d, p) implies divides_both(d, q)
}

theorem divides_both_step(a: Nat, p: Pair[Nat, Nat]) {
    divides_both(a, p) implies divides_both(a, gcd_step(p))
} by {
    if p.second = 0 {
    } else {
        a.divides(p.first)
        a.divides(p.second)
        a.divides(p.first.mod(p.second))
        Pair.new(p.second, p.first.mod(p.second)) = gcd_step(p)
        Pair.new(p.second, p.first.mod(p.second)).second = p.first.mod(p.second)
        a.divides(gcd_step(p).second)
        divides_both(a, gcd_step(p))
    }
}

theorem divides_gcd_step_n(d: Nat, p: Pair[Nat, Nat], n: Nat) {
    divides_both(d, p) implies divides_both(d, gcd_step_n(p, n))
} by {
    let f = function(k: Nat) {
        divides_both(d, p) implies divides_both(d, gcd_step_n(p, k))
    }
    gcd_step_n(p, Nat.zero) = p
    f(Nat.zero)
    forall(k: Nat) {
        if f(k) {
            if divides_both(d, p) {
                divides_both_step(d, gcd_step_n(p, k))
                divides_both(d, gcd_step(gcd_step_n(p, k)))
                divides_both(d, gcd_step_n(p, k.suc))
            }
            f(k.suc)
        }
    }
    f(n)
}

theorem divides_gcd_pair(d: Nat, p: Pair[Nat, Nat]) {
    divides_both(d, p) implies d.divides(gcd_pair(p))
}

theorem divides_gcd(d: Nat, a: Nat, b: Nat) {
    d.divides(a) and d.divides(b) implies d.divides(a.gcd(b))
} by {
    divides_both(d, Pair.new(a, b))
}

theorem divides_both_unstep(a: Nat, p: Pair[Nat, Nat]) {
    divides_both(a, gcd_step(p)) implies divides_both(a, p)
} by {
    if p.second = 0 {
    } else {
        Pair.new(p.second, p.first.mod(p.second)) = gcd_step(p)
        Pair.new(p.second, p.first.mod(p.second)).first = p.second
        a.divides(gcd_step(p).first)
        a.divides(p.second)
        a.divides(p.first.mod(p.second))
        a.divides(p.first)
        divides_both(a, p)
    }
}

theorem divides_gcd_step_n_converse(d: Nat, p: Pair[Nat, Nat], n: Nat) {
    divides_both(d, gcd_step_n(p, n)) implies divides_both(d, p)
} by {
    let f = function(x: Nat) {
        divides_both(d, gcd_step_n(p, x)) implies divides_both(d, p)
    }
    gcd_step_n(p, Nat.zero) = p
    f(0)
    forall(x: Nat) {
        if f(x) {
            if divides_both(d, gcd_step_n(p, x.suc)) {
                divides_both(d, gcd_step_n(p, x))
                divides_both(d, p)
            }
            f(x.suc)
        }
    }
    f(n)
}

theorem divides_gcd_pair_converse(d: Nat, p: Pair[Nat, Nat]) {
    d.divides(gcd_pair(p)) implies divides_both(d, p)
}

theorem gcd_divides(d: Nat, a: Nat, b: Nat) {
    d.divides(a.gcd(b)) implies d.divides(a) and d.divides(b)
} by {
    let p = Pair[Nat, Nat].new(a, b)
    d.divides(b)
}

theorem gcd_divides_left(a: Nat, b: Nat) {
    a.gcd(b).divides(a)
}

theorem gcd_divides_right(a: Nat, b: Nat) {
    a.gcd(b).divides(b)
}

theorem gcd_is_gcd(a: Nat, b: Nat, d: Nat) {
    a != 0 and b != 0 and d.divides(a) and d.divides(b) implies d <= a.gcd(b)
}

theorem gcd_nonzero_left(a: Nat, b: Nat) {
    a != 0 implies a.gcd(b) != 0
}

theorem gcd_zero_right(a: Nat) { a.gcd(0) = a }

theorem gcd_zero_left(a: Nat) { 0.gcd(a) = a }

theorem gcd_comm(a: Nat, b: Nat) { a.gcd(b) = b.gcd(a) } by {
    a.gcd(b).divides(b.gcd(a))
}

theorem gcd_nonzero_right(a: Nat, b: Nat) { b != 0 implies a.gcd(b) != 0 }

attributes Nat {
    /// True if this number and b have no common factor greater than one.
    define coprime(self, b: Nat) -> Bool { self.gcd(b) = 1 }
}

define mod_maintains(f: Nat -> Bool) -> Bool {
    forall(a: Nat, b: Nat) { f(a) and f(b) implies f(a.mod(b)) }
}

theorem mod_maintains_imp_gcd(f: Nat -> Bool, a: Nat, b: Nat) {
    mod_maintains(f) and f(a) and f(b) implies f(a.gcd(b))
} by {
    let p = Pair[Nat, Nat].new(a, b)
    let g = function(n: Nat) {
        f(gcd_step_n(p, n).first) and f(gcd_step_n(p, n).second)
    }
    f(gcd_step_n(p, 0).first)
    g(0)
    forall(x: Nat) {
        if g(x) {

            if gcd_step_n(p, x).second = 0 {
                f(gcd_step_n(p, x.suc).second)
                g(x.suc)
            } else {
                let q = gcd_step_n(p, x)
                g(x) = (f(q.first) and f(q.second))
                f(q.first)
                f(q.second)
                mod_maintains(f) = forall(j: Nat, k: Nat) {
                    f(j) and f(k) implies f(j.mod(k))
                }
                f(q.first) and f(q.second) implies f(q.first.mod(q.second))
                f(q.first.mod(q.second))
                q.first = gcd_step_n(p, x).first
                q.second = gcd_step_n(p, x).second
                gcd_step_n(p, x.suc).first = gcd_step_n(p, x).second
                f(gcd_step_n(p, x).first.mod(gcd_step_n(p, x).second))
                f(gcd_step_n(p, x.suc).second)
                g(x.suc)
            }
        }
    }
    f(gcd_step_n(p, num_gcd_steps(p)).first)
}

theorem gcd_one_right(a: Nat) {
    a.gcd(1) = 1
}

theorem gcd_one_left(a: Nat) {
    1.gcd(a) = 1
}

theorem gcd_mult_left_nonzero(a: Nat, b: Nat, m: Nat) {
    m != 0 implies m * a.gcd(b) = (m * a).gcd(m * b)
} by {
    m.divides(m * a)
    m.divides(m * b)
    m.divides((m * a).gcd(m * b))
    let d: Nat satisfy {
        m * d = (m * a).gcd(m * b)
    }

    // Overall we will prove equality by proving both sides divide the other.
    // First we prove that d equals gcd(a, b), to prove that right divides left.
    (m * a).gcd(m * b).divides(m * a)
    (m * d).divides(m * a)
    d.divides(a)
    d.divides(b)
    d.divides(a.gcd(b))

    // Now we prove left divides right.
    (m * a.gcd(b)).divides(m * b)
    (m * a.gcd(b)).divides((m * a).gcd(m * b))
}

theorem gcd_mult_left(a: Nat, b: Nat, m: Nat) {
    m * a.gcd(b) = (m * a).gcd(m * b)
} by {
    if m = 0 {
    } else {
    }
}

theorem gcd_mult_right(a: Nat, b: Nat, m: Nat) {
    a.gcd(b) * m = (a * m).gcd(b * m)
} by {
}

theorem cofactor(a: Nat, b: Nat, af: Nat, bf: Nat) {
    (
        a.gcd(b) != 0 and
        af * a.gcd(b) = a and
        bf * a.gcd(b) = b
    ) implies af.gcd(bf) = 1
} by {
    if a.gcd(b) != 0 and af * a.gcd(b) = a and bf * a.gcd(b) = b {
        let g = a.gcd(b)
        gcd_mult_left(af, bf, g)
        g * af.gcd(bf) = g * Nat.1
        mul_cancel_left(g, af.gcd(bf), Nat.1)
        af.gcd(bf) = Nat.1
    }
}

theorem gcd_of_prime(p: Nat, n: Nat) {
    p.is_prime implies p.gcd(n) = 1 or p.divides(n)
} by {
    p.gcd(n).divides(p)
    let d: Nat satisfy {
        d * p.gcd(n) = p
    }
    d != 0
    if d = 1 {
        p.gcd(n).divides(n)
        p.divides(n)
    } else {
        p.gcd(n) != 0
        p.gcd(n) = 1
    }
}
