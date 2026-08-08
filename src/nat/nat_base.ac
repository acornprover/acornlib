from algebra.add import Add
from data.basic.functions import Inhabited
from lte import LTE
from algebra.mul import Mul
from order import PartialOrder, LinearOrder, is_monotone, is_order_embedding, is_strict_monotone,
    order_embedding_is_monotone, order_embedding_is_strict_monotone
from data.basic.relation_basic import is_reflexive, is_transitive, is_antisymmetric

/// Natural numbers, the soul of arithmetic.
/// We build natural numbers from Acorn's inherent properties of inductive types.
inductive Nat {
    /// Zero is a natural number, because it's much more convenient this way.
    zero

    /// The successor of a natural number is also a natural number.
    suc(Nat)
}

from algebra.zero import Zero

instance Nat: Zero {
    let 0: Nat = Nat.zero
}

from algebra.one import One

instance Nat: One {
    let 1: Nat = Nat.0.suc
}

instance Nat: Inhabited {
    let default: Nat = Nat.0
}

attributes Nat {
    let 2: Nat = Nat.1.suc
    let 3: Nat = Nat.2.suc
    let 4: Nat = Nat.3.suc
    let 5: Nat = Nat.4.suc
    let 6: Nat = Nat.5.suc
    let 7: Nat = Nat.6.suc
    let 8: Nat = Nat.7.suc
    let 9: Nat = Nat.8.suc
    let 10: Nat = Nat.9.suc
}

/// Addition is defined recursively.
instance Nat: Add {
    define add(self, other: Nat) -> Nat {
        match other {
            Nat.zero {
                self
            }
            Nat.suc(pred) {
                Add.add[Nat](self, pred).suc
            }
        }
    }
}

numerals Nat

// Now let's have some theorems.

theorem add_zero_right(a: Nat) { a + 0 = a }

theorem one_plus_one { 1 + 1 = 2  }

theorem add_zero_left(a: Nat) { 0 + a = a } by {
    define f(x: Nat) -> Bool { 0 + x = x }
    0 + 0 = 0
    forall(x: Nat) {
        if f(x) {
            (0 + x).suc = 0 + x.suc
            f(x.suc)
        }
    }
}

theorem add_suc_right(a: Nat, b: Nat) {
    a + b.suc = (a + b).suc
}

theorem add_suc_left(a: Nat, b: Nat) { a.suc + b = (a + b).suc } by {
    define f(x: Nat) -> Bool { a.suc + x = (a + x).suc }
    a.suc + 0 = (a + 0).suc
    forall(x: Nat) {
        if f(x) {
            a.suc + x = (a + x).suc
            f(x.suc)
        }
    }
    f(b)
}

theorem add_one_right(a: Nat) {
    a + 1 = a.suc
}

theorem add_one_left(a: Nat) {
    1 + a = a.suc
}

theorem suc_ne(a: Nat) { a.suc != a } by {
    define f(x: Nat) -> Bool { x.suc != x }
    0.suc != 0
    forall(x: Nat) {
        if f(x) {
            if x.suc = x {
                false
            }
            f(x.suc)
        }
    }
}

theorem suc_suc_ne(a: Nat) { a.suc.suc != a } by {
    define f(x: Nat) -> Bool { x.suc.suc != x }
    f(0)
    forall(x: Nat) {
        if f(x) {
            f(x.suc)
        }
    }
}
 
theorem add_comm(a: Nat, b: Nat) { a + b = b + a } by {
    define f(x: Nat) -> Bool { x + b = b + x }
    f(0)
    forall(x: Nat) {
        if f(x) {
            x.suc + b = (x + b).suc
            b + x.suc = (b + x).suc
            f(x.suc)
        }
    }
}

theorem add_assoc(a: Nat, b: Nat, c: Nat) { a + b + c = a + (b + c) } by {
    define f(x: Nat) -> Bool { x + b + c = x + (b + c) }
    forall(x: Nat) {
        if f(x) {
            (x + b + c).suc = (x + b).suc + c
            x + (b + c) = x + b + c implies f(x.suc)
            f(x.suc)
        }
    }
    f(0)
}

// We define some "alt" versions of theorems to make it less annoying that Nat.zero and Nat.0
// are different symbols. 
theorem alt_induction(p: Nat -> Bool) {
    (p(Nat.0) and forall(n: Nat) { p(n) implies p(n.suc) })
    implies
    forall(n: Nat) { p(n) }                                                    
}    

theorem alt_suc_ne_zero(n: Nat) {
    n.suc != Nat.0
}

/// Multiplication is defined recursively.
instance Nat: Mul {
    define mul(self, b: Nat) -> Nat {
        match b {
            Nat.zero {
                0
            }
            Nat.suc(pred) {
                Mul.mul[Nat](self, pred) + self
            }
        }
    }
}

attributes Nat {
    /// The number formed by appending a digit to this one in base 10.
    define read(self, other: Nat) -> Nat {
        10 * self + other
    }
}

theorem mul_zero_right(a: Nat) { a * 0 = 0 }

theorem mul_zero_left(a: Nat) { 0 * a = 0 } by {
    define f(x: Nat) -> Bool { 0 * x = 0 }
    f(0)
    forall(x: Nat) {
        if f(x) {
            0 * x + 0 = 0 * x.suc
            0 + 0 = 0
            f(x.suc)
        }
    }
}

theorem mul_suc_right(a: Nat, b: Nat) {
    a * b.suc = a + a * b
}

theorem mul_suc_left(a: Nat, b: Nat) { a.suc * b = b + a * b } by {
    define f(x: Nat) -> Bool { a.suc * x = x + a * x }
    a.suc * 0 = 0 + a * 0
    forall(x: Nat) {
        if f(x) {
            x + a * x = a.suc * x
            a + a * x = a * x + a
            x.suc + (a + a * x) = (x + (a + a * x)).suc
            a.suc + a.suc * x = (a + a.suc * x).suc
            a + a.suc * x = a.suc * x + a
            a.suc * x.suc = x.suc + (a + a * x)
            f(x.suc)
        }
    }
    f(b)
}

theorem mul_one_one { 1 * 1 = 1 }

theorem mul_comm(a: Nat, b: Nat) { a * b = b * a } by {
    define f(x: Nat) -> Bool { x * b = b * x }
    0 * b = b * 0
    forall(x: Nat) {
        if f(x) {
            b + x * b = b + b * x
            f(x.suc)
        }
    }
    f(a)
}

theorem add_comm_4(a: Nat, b: Nat, c: Nat, d: Nat) {
    (a + b) + (c + d) = (a + c) + (b + d)
}

theorem distrib_left(a: Nat, b: Nat, c: Nat) {
    a * (b + c) = a * b + a * c
} by {
    define f(x: Nat) -> Bool { x * (b + c) = x * b + x * c }
    f(0)
    forall(x: Nat) {
        if f(x) {
            b + c + x * (b + c) = b + c + (x * b + x * c)
            x.suc * (b + c) = x.suc * b + x.suc * c
            f(x.suc)
        }
    }
}

theorem distrib_right(a: Nat, b: Nat, c: Nat) {
    (a + b) * c = a * c + b * c
}

theorem mul_assoc(a: Nat, b: Nat, c: Nat) { a * b * c = a * (b * c) } by {
    define f(x: Nat) -> Bool { x * b * c = x * (b * c) }
    0 * b = 0
    (0 * b) * c = 0 * c
    0 * c = 0
    0 * (b * c) = 0
    0 * b * c = 0
    0 * (b * c) = 0
    0 * b * c = 0 * (b * c)
    f(0)
    forall(x: Nat) {
        if f(x) {
            x * (b * c) = x * b * c
            (b + x * b) * c = b * c + x * b * c
            f(x.suc)
        }
    }
}

theorem mul_one_right(a: Nat) { a * 1 = a }

theorem mul_one_left(a: Nat) { 1 * a = a }

/// Multiplying a natural number by two gives its double.
theorem mul_two_left(a: Nat) { Nat.2 * a = a + a } by {
    mul_suc_left(Nat.1, a)
    mul_one_left(a)
}

theorem add_cancels_left(a: Nat, b: Nat, c: Nat) { a + b = a + c implies b = c } by {
    define f(x: Nat) -> Bool { add_cancels_left(x, b, c) }
    if not add_cancels_left(0, b, c) {
        false
    }
    forall(x: Nat) {
        if f(x) {
            f(x.suc)
        }
    }
}

// Ordering

/// `a <= b` if there's a natural number that can be added to `a` to get `b`.
instance Nat: LTE {
    define lte(self, b: Nat) -> Bool {
        exists(c: Nat) {
            self + c = b
        }
    }
}

theorem lte_ref(a: Nat) {
    a <= a
}

theorem nat_is_reflexive {
    is_reflexive(Nat.lte)
}

theorem lte_trans(a: Nat, b: Nat, c: Nat) { a <= b and b <= c implies a <= c } by {
    if a <= b and b <= c {
        let d1: Nat satisfy { a + d1 = b }
        let d2: Nat satisfy { b + d2 = c }
        a + (d1 + d2) = c
        a <= c
    }
    define f(z: Nat) -> Bool {
        forall(x: Nat, y: Nat) { lte_trans(x, y, z) }
    }
    forall(x: Nat, y: Nat) {
        if x <= y and y <= 0 {
            let d1: Nat satisfy { x + d1 = y }
            let d2: Nat satisfy { y + d2 = Nat.0 }
            (d1 + d2) + x = x + (d1 + d2)
            if x != Nat.0 {
                let k: Nat satisfy { x = k.suc }
                (d1 + d2 + k).suc != Nat.0
                false
            }
            x <= 0
        }
    }
}

theorem nat_is_transitive {
    is_transitive(Nat.lte)
} by {
    forall(x: Nat, y: Nat, z: Nat) {
    }
}

theorem add_to_zero(a: Nat, b: Nat) { a + b = 0 implies a = 0 and b = 0 } by {
    define f(x: Nat) -> Bool { x + b = 0 implies x = 0 and b = 0 }
    forall(x: Nat) {
        f(x.suc)
    }
    f(0)
    f(a)
}

theorem lte_antisymm(a: Nat, b: Nat) {
    a <= b and b <= a implies a = b  
} by {
    let c: Nat satisfy {
        a + c = b
    }
    let d: Nat satisfy {
        b + d = a
    }
    d = 0
}

theorem nat_is_antisymmetric {
    is_antisymmetric(Nat.lte)
}

instance Nat: PartialOrder

theorem lt_not_ref(a: Nat) {
    not (a < a)
}

theorem only_zero_lte_zero(a: Nat) {
    a <= 0 implies a = 0
}

theorem not_lt_zero(a: Nat) {
    not a < 0
}

theorem alt_not_lt_zero(a: Nat) {
    not a < Nat.zero
}

theorem zero_or_suc(a: Nat) {
    a = 0 or exists(b: Nat) { a = b.suc }
}

theorem lte_cancel_suc(a: Nat, b: Nat) { a.suc <= b.suc implies a <= b }

/// The successor operation preserves the non-strict natural order.
theorem lte_suc_suc(a: Nat, b: Nat) {
    a <= b implies a.suc <= b.suc
} by {
    if a <= b {
        let d: Nat satisfy { a + d = b }
        a.suc + d = b.suc
        a.suc <= b.suc
    }
}

theorem lt_cancel_suc(a: Nat, b: Nat) {
    a.suc < b.suc implies a < b
}

theorem lt_not_symm(a: Nat, b: Nat) { a < b implies not b < a } by {
    define f(x: Nat) -> Bool {
        forall(y: Nat) { lt_not_symm(x, y) }
    }
}

theorem lt_diff(a: Nat, b: Nat) {
    a < b implies exists(c: Nat) { a + c = b and c != 0 }
} by {
    if a < b {
        a <= b
        let c: Nat satisfy { a + c = b }
        if c = 0 {
            a + 0 < b
            false
        }
        exists(d: Nat) { a + d = b and d != 0 }
    }
}

theorem lt_and_lte(a: Nat, b: Nat, c: Nat) { a < b and b <= c implies a < c }

theorem lte_and_lt(a: Nat, b: Nat, c: Nat) { a <= b and b < c implies a < c }

theorem lt_trans(a: Nat, b: Nat, c: Nat) { a < b and b < c implies a < c }

theorem add_cancels_right(a: Nat, b: Nat, c: Nat) { b + a = c + a implies b = c }

theorem add_identity_right(a: Nat, b: Nat) {
    a + b = a implies b = 0
}

theorem lt_add_suc(a: Nat, b: Nat) { a < a + b.suc }

theorem lt_suc(a: Nat) { a < a.suc }

theorem lt_suc_left(a: Nat, b: Nat) { a < b implies a.suc = b or a.suc < b } by {
    let (c: Nat) satisfy { a + c = b and c != 0 }
    let (d: Nat) satisfy { d.suc = c }
    if d = 0 {
        a.suc = b
    } else {
        let (e: Nat) satisfy { e.suc = d }
        a.suc + d = (a + d).suc
        a.suc + e.suc = b
        a.suc < b
    }
}

theorem lt_suc_right(a: Nat, b: Nat) { a < b.suc implies a = b or a < b }

theorem lt_add_left(a: Nat, b: Nat, c: Nat) { b < c implies a + b < a + c } by {
    if b < c {
        let (d: Nat) satisfy { b + d = c and d != 0 }
        let (e: Nat) satisfy { e.suc = d }
        a + b + e.suc = a + c
        a + b < a + c
    }
}

theorem trichotomy(a: Nat, b: Nat) { a < b or b < a or a = b } by {
    define f(x: Nat) -> Bool { a < x or x < a or a = x }
    f(0)
    forall(x: Nat) {
        if f(x) {
            x < x.suc
            if x < a {
                f(x.suc)
            }
            if a < x {
                a <= x
                a < x.suc
                f(x.suc)
            }
            if x = a {
                f(x.suc)
            }
            f(x.suc)
        }
    }
}

theorem lt_or_lte(a: Nat, b: Nat) { a < b or b <= a }

theorem lt_imp_lte_suc(a: Nat, b: Nat) { a < b implies a.suc <= b }

theorem lte_imp_not_lt(a: Nat, b: Nat) { a <= b implies not (b < a) } by {
    if a = b {
    } else {
    }
}

theorem division_theorem(m: Nat, n: Nat) {
    0 < n implies exists(q: Nat, r: Nat) {
        r < n and m = q * n + r
    }
} by {
    define f(x: Nat) -> Bool { division_theorem(x, n) }
    if 0 < n {
        0 < n and 0 = 0 * n + 0
        exists(q: Nat, r: Nat) { r < n and 0 = q * n + r }
    }
    f(0)
    forall(x: Nat) {
        if f(x) {
            let (q: Nat, r: Nat) satisfy {
                r < n and x = q * n + r
            }
            if r.suc = n {
            } else {
                f(x.suc)
            }
            f(x.suc)
        }
    }
    f(m)
    if 0 < n {
        0 = 0 * n + 0
    }
}

attributes Nat {
    /// True if this number is composite (has nontrivial factors).
    define is_composite(self) -> Bool {
        exists(b: Nat, c: Nat) {
            1 < b and 1 < c and self = b * c
        }
    }

    /// True if this number is prime (greater than 1 and not composite).
    define is_prime(self) -> Bool {
        1 < self and not self.is_composite
    }

    /// True if this number divides b (equivalently, there exists c such that this * c = b).
    define divides(self, b: Nat) -> Bool {
        exists(c: Nat) { self * c = b }
    }

    /// The factorial of this number (the product 1 * 2 * ... * n).
    define factorial(self) -> Nat {
        match self {
            Nat.zero {
                1
            }
            Nat.suc(pred) {
                self * pred.factorial
            }
        }
    }
}

theorem mul_to_zero(a: Nat, b: Nat) { a * b = 0 implies a = 0 or b = 0 } by {
    match b {
        Nat.zero {
        }
        Nat.suc(pred) {
            a * pred.suc = a + a * pred
            if a * b = 0 {
                a + a * pred = 0
                a = 0
            }
        }
    }
}

theorem divisor_lt(a: Nat, b: Nat, c: Nat) {
    a != 0 and 1 < b and a * b = c implies a < c
} by {
    let (d: Nat) satisfy { 1 + d = b and d != 0 }
    a * d != 0
    a <= c
}

theorem divides_self(a: Nat) { a.divides(a) }

define true_below(f: Nat -> Bool, n: Nat) -> Bool {
    forall(x: Nat) { x < n implies f(x) }
}

/// A predicate true below `n` is true at each smaller number.
theorem true_below_apply(f: Nat -> Bool, n: Nat, x: Nat) {
    true_below(f, n) and x < n implies f(x)
} by {
    if true_below(f, n) and x < n {
        true_below(f, n) = forall(y: Nat) {
            y < n implies f(y)
        }
        f(x)
    }
}

/// Every predicate is true below zero.
theorem true_below_zero(f: Nat -> Bool) {
    true_below(f, Nat.0)
} by {
}

/// If a predicate holds below `n` and at `n`, then it holds below `n.suc`.
theorem true_below_suc_intro(f: Nat -> Bool, n: Nat) {
    true_below(f, n) and f(n) implies true_below(f, n.suc)
} by {
    if true_below(f, n) and f(n) {
        forall(x: Nat) {
            if x < n.suc {
                if x = n {
                    f(x)
                } else {
                    true_below_apply(f, n, x)
                    f(x)
                }
            }
        }
        true_below(f, n.suc)
    }
}

theorem strong_induction(f: Nat -> Bool) {
    forall(k: Nat) {
        true_below(f, k) implies f(k)
    } implies forall(n: Nat) { f(n) }
} by {
    define g(x: Nat) -> Bool {
        true_below(f, x)
    }
    true_below(f, 0)
    g(0)
    forall(x: Nat) {
        if g(x) {
            true_below(f, x)
            forall(y: Nat) {
                if y < x.suc {
                    y = x or y < x
                    if y = x {
                        f(y)
                    } else {
                        y < x
                        f(y)
                    }
                }
            }
            g(x.suc)
        }
    }
    forall(n: Nat) {
        g(n)
    }
    forall(n: Nat) {
        f(n)
    }
    forall(n: Nat) {
        f(n)
    }
}

theorem divides_trans(a: Nat, b: Nat, c: Nat) {
    a.divides(b) and b.divides(c) implies a.divides(c)
} by {
    if a.divides(b) and b.divides(c) {
        let d1: Nat satisfy { a * d1 = b }
        let d2: Nat satisfy { b * d2 = c }
        a * (d1 * d2) = c
        a.divides(c)
    }
}

theorem has_prime_divisor(n: Nat) {
    1 < n implies exists(p: Nat) {
        p.is_prime and p.divides(n)
    }
} by {
    forall(k: Nat) {
        if true_below(has_prime_divisor, k) {
            if k.is_prime {
                1 < k implies exists(p: Nat) {
                    p.is_prime and p.divides(k)
                }
            } else {
                if k <= 1 {
                    has_prime_divisor(k)
                } else {
                    1 < k
                    k.is_composite
                    let (b: Nat, c: Nat) satisfy {
                        1 < b and 1 < c and k = b * c
                    }
                    b < k
                    has_prime_divisor(b)
                    let p: Nat satisfy {
                        p.is_prime and p.divides(b)
                    }
                    p.divides(k)
                    exists(q: Nat) {
                        q.is_prime and q.divides(k)
                    }
                }
            }
        }
    }
    1 < n implies exists(p: Nat) {
        p.is_prime and p.divides(n)
    }
}

theorem factorial_zero {
    0.factorial = 1
}

theorem factorial_one {
    1.factorial = 1
}

theorem factorial_step(n: Nat) {
    n.suc.factorial = n.suc * n.factorial
}

theorem divides_factorial(k: Nat, n: Nat) {
    k != 0 and k <= n implies k.divides(n.factorial)
} by {
    define f(x: Nat) -> Bool {
        k != 0 and k <= x implies k.divides(x.factorial)
    }
    f(0)
    forall(x: Nat) {
        if f(x) {
            if k <= x.suc {
                if k = x.suc {
                    k.divides(k.factorial)
                    k.divides(x.suc.factorial)
                } else {
                    not (x.suc < k)
                    k <= x
                    k != 0 and k <= x
                    k.divides(x.factorial)
                    x.factorial.divides(x.suc.factorial)
                    k.divides(x.suc.factorial)
                }
            }
            f(x.suc)
        }
    }
    f(n)
}

theorem factorial_nondecreasing(n: Nat) { n.factorial <= n.suc.factorial }

theorem lte_one_factorial(a: Nat) { 1 <= a.factorial } by {
    1 <= 0.factorial
    forall(x: Nat) {
        if lte_one_factorial(x) {
            1 <= x.suc.factorial
        }
    }
}

theorem lt_imp_lt_suc(a: Nat, b: Nat) { a < b implies a < b.suc }

theorem lte_mul_both(a: Nat, b: Nat, c: Nat) { b <= c implies a * b <= a * c } by {
    if b <= c {
        let d: Nat satisfy { b + d = c }
        a * b + a * d = a * c
        a * b <= a * c
    }
}

theorem lt_mul_both(a: Nat, b: Nat, c: Nat) { a != 0 and b < c implies a * b < a * c } by {
    let (d: Nat) satisfy { b + d = c }
    if a * b = a * c {
        let y: Nat = a * b
        let z: Nat = a * d
        y + z = y
        false
    }
}

theorem lt_cancel_mul(a: Nat, b: Nat, c: Nat) { a != 0 and a * b < a * c implies b < c }

theorem mul_to_one(a: Nat, b: Nat) { a * b = 1 implies a = 1 } by {
    if 1 < a {
        b = 0
        false
    }
}

theorem divides_suc(a: Nat, b: Nat) { a.divides(b) and a.divides(b.suc) implies a = 1 } by {
    let (c: Nat) satisfy { a * c = b }
    let (d: Nat) satisfy { a * d = b.suc }
    if a = 0 {
        0 * d = 0
        b.suc = 0
        false
    }
    a * c < a * d
    let (e: Nat) satisfy { c + e = d and e != 0 }
    a * c + a * e = b.suc
    b + a * e = b + 1
    a * e = 1
}

/// Divisibility by two alternates between a natural number and its successor.
theorem two_divides_suc_iff(n: Nat) {
    Nat.2.divides(n.suc) = not Nat.2.divides(n)
} by {
    if Nat.2.divides(n) {
        if Nat.2.divides(n.suc) {
            divides_suc(Nat.2, n)
            false
        }
        Nat.2.divides(n.suc) = false
        not Nat.2.divides(n) = false
        Nat.2.divides(n.suc) = not Nat.2.divides(n)
    }
    if not Nat.2.divides(n) {
        division_theorem(n, Nat.2)
        let (q: Nat, r: Nat) satisfy {
            r < Nat.2 and n = q * Nat.2 + r
        }
        if r != Nat.0 and r != Nat.1 {
            lt_suc_right(r, Nat.1)
            r < Nat.1
            lt_suc_right(r, Nat.0)
            not_lt_zero(r)
            false
        }
        r = Nat.0 or r = Nat.1
        if r = Nat.0 {
            n = q * Nat.2
            q * Nat.2 = Nat.2 * q
            n = Nat.2 * q
            Nat.2.divides(n)
            false
        }
        if r = Nat.1 {
            n.suc = Nat.2 * q + Nat.2
            n.suc = Nat.2 * q.suc
            Nat.2.divides(n.suc)
        }
        Nat.2.divides(n.suc) = true
        not Nat.2.divides(n) = true
        Nat.2.divides(n.suc) = not Nat.2.divides(n)
    }
}

theorem exists_infinite_primes(n: Nat) {
    exists(p: Nat) {
        n < p and p.is_prime
    }
} by {
    let m: Nat = n.factorial.suc
    1 < m
    let (p: Nat) satisfy {
        p.is_prime and p.divides(m)
    }
    n < p
}

theorem divides_zero(a: Nat) {
    a.divides(0)
}

theorem zero_divides(a: Nat) { 0.divides(a) implies a = 0 }

theorem divides_mul(a: Nat, b: Nat, d: Nat) { d.divides(a) implies d.divides(a * b) }

theorem lte_mul(a: Nat, b: Nat) { b != 0 implies a <= a * b }

theorem divides_lte(a: Nat, b: Nat) { a.divides(b) implies b = 0 or a <= b } by {
    if a.divides(b) {
        let q: Nat satisfy { q * a = b }
        if q = 0 {
            0 * a = 0
            b = 0 or a <= b
        } else {
            a <= a * q
            a <= b
            b = 0 or a <= b
        }
    }
}

theorem divides_add(a: Nat, b: Nat, d: Nat) {
    d.divides(a) and d.divides(b) implies d.divides(a + b)
} by {
    let (qa: Nat) satisfy { qa * d = a }
    let (qb: Nat) satisfy { qb * d = b }
    a + b = d * (qa + qb)
}

theorem divides_symm(a: Nat, b: Nat) { a.divides(b) and b.divides(a) implies a = b } by {
    if a = 0 {
    } else {
        a = b
    }
}

theorem cross_sum_lte(a: Nat, b: Nat, c: Nat, d: Nat) { a + b = c + d and a <= c implies d <= b } by {
    let (e: Nat) satisfy { a + e = c }
    a + (e + d) = a + e + d
    e + d = d + e
}

theorem sum_lte(a: Nat, b: Nat, c: Nat, d: Nat) { a <= c and b <= d implies a + b <= c + d } by {
    let (e: Nat) satisfy { a + e = c }
    let (f: Nat) satisfy { b + f = d }
    a + e + (b + f) = a + b + (e + f)
}

theorem lte_add_left(a: Nat, b: Nat, c: Nat) {
    b <= c implies a + b <= a + c
} by {
    if b <= c {
        lte_ref(a)
        sum_lte(a, b, a, c)
        a + b <= a + c
    }
}

theorem lte_add_right(a: Nat, b: Nat, c: Nat) {
    b <= c implies b + a <= c + a
} by {
    if b <= c {
        lte_add_left(a, b, c)
        b + a <= c + a
    }
}

theorem lte_mul_left(a: Nat, b: Nat, c: Nat) {
    b <= c implies a * b <= a * c
} by {
    if b <= c {
        lte_mul_both(a, b, c)
        a * b <= a * c
    }
}

theorem lte_mul_right(a: Nat, b: Nat, c: Nat) {
    b <= c implies b * a <= c * a
} by {
    if b <= c {
        lte_mul_left(a, b, c)
        a * b <= a * c
        b * a <= c * a
    }
}

/// The function that adds a fixed natural number on the left.
define nat_add_left_map(a: Nat, n: Nat) -> Nat {
    a + n
}

/// The function that adds a fixed natural number on the right.
define nat_add_right_map(a: Nat, n: Nat) -> Nat {
    n + a
}

/// The function that multiplies by a fixed natural number on the left.
define nat_mul_left_map(a: Nat, n: Nat) -> Nat {
    a * n
}

/// The function that multiplies by a fixed natural number on the right.
define nat_mul_right_map(a: Nat, n: Nat) -> Nat {
    n * a
}

/// Successor is an order embedding of the natural numbers.
theorem nat_suc_is_order_embedding {
    is_order_embedding(Nat.suc)
} by {
    forall(x: Nat, y: Nat) {
        if x.suc <= y.suc {
            lte_cancel_suc(x, y)
            x <= y
        }
        if x <= y {
            lte_suc_suc(x, y)
            x.suc <= y.suc
        }
        x.suc <= y.suc = (x <= y)
    }
}

/// Successor is monotone on the natural numbers.
theorem nat_suc_is_monotone {
    is_monotone(Nat.suc)
} by {
    nat_suc_is_order_embedding
    order_embedding_is_monotone(Nat.suc)
}

/// Successor is strictly monotone on the natural numbers.
theorem nat_suc_is_strict_monotone {
    is_strict_monotone(Nat.suc)
} by {
    nat_suc_is_order_embedding
    order_embedding_is_strict_monotone(Nat.suc)
}

/// Adding a fixed natural number on the left is monotone.
theorem nat_add_left_map_is_monotone(a: Nat) {
    is_monotone(nat_add_left_map(a))
} by {
    forall(x: Nat, y: Nat) {
        if x <= y {
            lte_add_left(a, x, y)
            nat_add_left_map(a, x) <= nat_add_left_map(a, y)
        }
    }
}

/// Adding a fixed natural number on the right is monotone.
theorem nat_add_right_map_is_monotone(a: Nat) {
    is_monotone(nat_add_right_map(a))
} by {
    forall(x: Nat, y: Nat) {
        if x <= y {
            lte_add_right(a, x, y)
            nat_add_right_map(a, x) <= nat_add_right_map(a, y)
        }
    }
}

/// Multiplying by a fixed natural number on the left is monotone.
theorem nat_mul_left_map_is_monotone(a: Nat) {
    is_monotone(nat_mul_left_map(a))
} by {
    forall(x: Nat, y: Nat) {
        if x <= y {
            lte_mul_left(a, x, y)
            nat_mul_left_map(a, x) <= nat_mul_left_map(a, y)
        }
    }
}

/// Multiplying by a fixed natural number on the right is monotone.
theorem nat_mul_right_map_is_monotone(a: Nat) {
    is_monotone(nat_mul_right_map(a))
} by {
    forall(x: Nat, y: Nat) {
        if x <= y {
            lte_mul_right(a, x, y)
            nat_mul_right_map(a, x) <= nat_mul_right_map(a, y)
        }
    }
}

// This is a "bounded" version of subtraction that returns 0 instead of negative numbers.
let bounded_sub(a: Nat, b: Nat) -> d: Nat satisfy {
    if a < b {
        d = 0
    } else {
        d + b = a
    }
} by {
    if a < b {
        let x: Nat satisfy {
            x = 0
        }
    } else {
        let x: Nat satisfy {
            b + x = a
        }
        x + b = a
    }
}

attributes Nat {
    /// Subtraction on natural numbers is defined oddly; it "caps out" at zero.
    /// If `self < b`, then `self - b = 0`.
    /// It would be better to define this as "not valid" on some inputs, but
    /// the language doesn't make that convenient yet.
    define sub(self, b: Nat) -> Nat { bounded_sub(self, b) }
}

theorem sub_lt(a: Nat, b: Nat) { a < b implies a - b = 0 }

theorem add_sub(a: Nat, b: Nat) {
    b <= a implies a - b + b = a
}

theorem sub_add(a: Nat, b: Nat) { (a + b) - b = a }

theorem sub_self(a: Nat) { a - a = 0 }

theorem sub_zero(a: Nat) {
    a - 0 = a
}

theorem alt_add_zero(a: Nat, b: Nat) {
    b != Nat.0 or a + b = a
}

theorem add_imp_sub(a: Nat, b: Nat, c: Nat) { a + b = c implies c - b = a }

theorem add_imp_sub_left(a: Nat, b: Nat, c: Nat) {
    a + b = c implies c - a = b
} by {
    if a + b = c {
        add_imp_sub(b, a, c)
        c - a = b
    }
}

/// If a > b, then a - b > 0.
theorem sub_pos(a: Nat, b: Nat) {
    a > b implies a - b > 0
} by {
    if a > b {
        let c: Nat satisfy { b + c = a and c != 0 }
        if a - b = 0 {
            false
        }
        a - b > 0
    }
}

/// The successor of a number minus one equals the number.
theorem suc_sub_one(a: Nat) {
    a.suc - 1 = a
}

/// When n > 0, subtracting 1 gives a value less than n.
theorem sub_one_lt(n: Nat) {
    n > 0 implies n - 1 < n
} by {
    if n > 0 {
        let m: Nat satisfy { m.suc = n }
        n - 1 = m
        m < m.suc
        n - 1 < n
    }
}

/// When n > 0, subtracting 1 and then any amount x gives a value less than n.
theorem sub_one_sub_lt(n: Nat, x: Nat) {
    n > 0 implies n - 1 - x < n
} by {
    if n > 0 {
        // First show n - 1 < n
        // For any x, n - 1 - x <= n - 1 < n (by bounded subtraction)
        // So n - 1 - x < n
        if x > n - 1 {
            // If x > n - 1, then n - 1 - x = 0 < n
        } else {
            // If x <= n - 1, then n - 1 - x <= n - 1 < n
            (n - 1 - x) + x = n - 1 or n - 1 < x
            n - 1 - x <= n - 1
            n - 1 < n
            n - 1 - x < n
        }
    }
}

/// Subtraction with successor: (a + 1) - b - 1 = a - b when b <= a.
theorem sub_suc_sub_one(a: Nat, b: Nat) {
    b <= a implies (a + 1) - b - 1 = a - b
} by {
    if b <= a {
        let d = (a + 1) - b

        // d >= 1 because a + 1 > b (since b <= a means a >= b, so a + 1 > b)
        if b = a {
        } else {
            a < a.suc
            a + 1 = a.suc
            b < a.suc
            a.suc - b > 0
            d >= 1
            d - 1 + 1 = d
            b <= a.suc
            a.suc - b + b = a.suc
            d - 1 + 1 + b = d - 1 + (1 + b)
            (d - 1) + (1 + b) = a + 1
            a - b = d - 1
        }

        (a + 1) - b - 1 = a - b
    }
}

/// Subtraction is commutative in the operands being subtracted: (a - b) - c = (a - c) - b when b + c <= a.
theorem sub_comm(a: Nat, b: Nat, c: Nat) {
    b + c <= a implies (a - b) - c = (a - c) - b
} by {
    if b + c <= a {
        // We'll show both sides equal a - (b + c)

        // First, show (a - b) - c = a - (b + c)
        let d1 = a - b
        let result = a - (b + c)

        // We need to show d1 - c = result
        // We have d1 + b = a and result + b + c = a
        // So d1 + b = result + b + c
        // Therefore d1 = result + c
        // So d1 - c = result

        d1 + b = result + (b + c)

        // Similarly, show (a - c) - b = a - (b + c) = a - (c + b)
        let d2 = a - c
        let result2 = a - (c + b)
        c + b = b + c
        a - (b + c) + (b + c) = a - (b + c) + b + c
        a - (c + b) + (c + b) = a - (c + b) + c + b
        a - (c + b) + b = a - c
        d2 + c = result2 + c + b
        (a - c) - b = a - (c + b)

        // Since b + c = c + b, we have a - (b + c) = a - (c + b)
        (a - b) - c = (a - c) - b
    }
}

let nat_mod(a: Nat, m: Nat) -> r: Nat satisfy {
    if m != 0 {
        r < m and exists(q: Nat) { q * m + r = a }
    } else {
        // It doesn't really matter how we define "mod 0".
        // We pick a mod 0 = a.
        r = a
    }
} by {
    if m != 0 {
        let (q: Nat, r: Nat) satisfy {
            r < m and a = q * m + r
        }
        let x: Nat satisfy {
            x < m and exists(q0: Nat) { q0 * m + x = a }
        }
    } else {
        let x: Nat satisfy {
            x = a
        }
    }
}

attributes Nat {
    /// The remainder when dividing this number by m.
    let mod = nat_mod
}

theorem add_mod(a: Nat, m: Nat) {
    exists(q: Nat) { q * m + a.mod(m) = a }
} by {
    if m = 0 {
        0 * 0 + a.mod(0) = a
    } else {
    }
}

theorem mod_by_zero(a: Nat) { a.mod(0) = a }

theorem mod_of_zero(m: Nat) {
    0.mod(m) = 0
} by {
    if m = 0 {
    } else {
        let q: Nat satisfy { q * m + 0.mod(m) = 0 }
        0.mod(m) = 0
    }
}

theorem mod_lte(a: Nat, m: Nat) {
    a.mod(m) <= a
} by {
    if m = 0 {
    } else {
        let q: Nat satisfy { q * m + a.mod(m) = a }
        a.mod(m) <= a
    }
}

theorem div_sub_mod(a: Nat, m: Nat) { m.divides(a - a.mod(m)) } by {
    let (q: Nat) satisfy { q * m + a.mod(m) = a }
}

theorem sub_left_distrib(a: Nat, b: Nat, c: Nat) {
    a * (b - c) = a * b - a * c
} by {
    if a = 0 {
    } else {
        if b < c {
            a * b < a * c
            a * b - a * c = 0
            b - c = 0
            a * 0 = 0
            a * (b - c) = a * b - a * c
        } else {
            a * (b - c) = a * b - a * c
        }
    }
}

theorem sub_right_distrib(a: Nat, b: Nat, c: Nat) {
    (a - b) * c = a * c - b * c
}

theorem divides_sub(a: Nat, b: Nat, d: Nat) {
    d.divides(a) and d.divides(b) implies d.divides(a - b)
} by {
    let qa: Nat satisfy { qa * d = a }
    let qb: Nat satisfy { qb * d = b }
    qa * d - qb * d = (qa - qb) * d
    d * (qa - qb) = (qa - qb) * d
}

theorem divides_mod(a: Nat, m: Nat, d: Nat) {
    d.divides(a) and d.divides(m) implies d.divides(a.mod(m))
} by {
    let (q: Nat) satisfy { q * m + a.mod(m) = a }
    d.divides(a - q * m)
}

theorem div_imp_mod(a: Nat, m: Nat) { m.divides(a) implies a.mod(m) = 0 } by {
    if m != 0 {
    } else {
        a.mod(m) = 0
    }
}

theorem small_mod(a: Nat, m: Nat) { a < m implies a.mod(m) = a } by {
    let (q: Nat) satisfy { q * m + a.mod(m) = a }
    if q = 0 {
    } else {
        false
    }
}

theorem mod_mod(a: Nat, m: Nat) { a.mod(m).mod(m) = a.mod(m) } by {
    if m != 0 {
    } else {
    }
}

theorem mod_mul(m: Nat, q: Nat) { (q * m).mod(m) = 0 }

theorem divides_add_copy(a: Nat, b: Nat, d: Nat) {
    d.divides(a) and d.divides(b) implies d.divides(a + b)
}

theorem divides_unmod(d: Nat, a: Nat, m: Nat) {
    d.divides(m) and d.divides(a.mod(m)) implies d.divides(a)
} by {
    if d.divides(m) and d.divides(a.mod(m)) {
        let q: Nat satisfy { q * m + a.mod(m) = a }
        d.divides(m * q)
        d.divides(q * m + a.mod(m))
        d.divides(a)
    }
}

theorem mul_cancel_left(a: Nat, b: Nat, c: Nat) {
    a != 0 and a * b = a * c implies b = c
} by {
    not b < c
}

theorem mul_cancel_right(a: Nat, b: Nat, c: Nat) {
    a != 0 and b * a = c * a implies b = c
}

theorem divides_cancel_left(a: Nat, b: Nat, c: Nat) {
    a != 0 and (a * b).divides(a * c) implies b.divides(c)
} by {
    if a != 0 and (a * b).divides(a * c) {
        let d: Nat satisfy { a * b * d = a * c }
        a * (b * d) = a * b * d
        b * d = c
        b.divides(c)
    }
}

theorem divides_cancel_right(a: Nat, b: Nat, c: Nat) {
    a != 0 and (b * a).divides(c * a) implies b.divides(c)
}

theorem divides_mul_left(a: Nat, b: Nat, m: Nat) {
    a.divides(b) implies (m * a).divides(m * b)
} by {
    if a.divides(b) {
        let d: Nat satisfy { a * d = b }
        m * a * d = m * b
        (m * a).divides(m * b)
    }
}

theorem divides_mul_right(a: Nat, b: Nat, m: Nat) {
    a.divides(b) implies (a * m).divides(b * m)
}

// Misc helpers
theorem two_neq_zero {
    2 != 0
}
theorem three_neq_zero {
    3 != 0
}

theorem gte_each_of_three(a: Nat, b: Nat, c: Nat) {
    exists(n: Nat) {
        a <= n and b <= n and c <= n
    }
} by {
    let n1: Nat satisfy {
        a <= n1 and b <= n1
    }
    let n2: Nat satisfy {
        n1 <= n2 and c <= n2
    }
}

theorem gte_each_of_three_regular(a: Nat, b: Nat, c: Nat) {
    exists(n: Nat) {
        n >= a and n >= b and n >= c
    }
} by {
    let n: Nat satisfy {
        a <= n and b <= n and c <= n
    }
}

theorem gt_each_of_three_swapped(a: Nat, b: Nat, c: Nat) {
    exists(n: Nat) {
        a < n and b < n and c < n
    }
} by {
    let n1: Nat satisfy {
        a <= n1 and b <= n1 and c <= n1
    }
}

theorem gt_each_of_three_regular(a: Nat, b: Nat, c: Nat) {
    exists(n: Nat) {
        n > a and n > b and n > c
    }
} by {
    let n: Nat satisfy {
        a < n and b < n and c < n
    }
}

// Single-digit addition facts should probably just be theorems.
theorem one_plus_two {
    1 + 2 = 3
}

theorem one_plus_three {
    1 + 3 = 4
}

theorem one_plus_four {
    1 + 4 = 5
}

theorem one_plus_five {
    1 + 5 = 6
}

instance Nat: LinearOrder
