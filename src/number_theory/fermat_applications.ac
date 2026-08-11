/// Fermat applications.
///
/// This file collects the classical applications of Fermat's little theorem
/// to concrete numbers:
///
///   (a) the Fermat test: for a prime `n` and a base `a` coprime to `n`, the
///       power `a^(n - 1)` is congruent to `1` modulo `n` (fermat.ac,
///       restated);
///   (b) the base-two Fermat pseudoprime `341 = 11 * 31`: `341` is composite
///       yet `2^340 ≡ 1 (mod 341)`. The congruence is verified modulo each of
///       the coprime factors `11` and `31` — by Fermat at `2^10 ≡ 1 (mod 11)`
///       and `2^30 ≡ 1 (mod 31)`, together with `2^10 ≡ 1 (mod 31)` from
///       `2^5 = 32 = 31 + 1` — and combined by the Chinese remainder theorem.
///       `341` is the first Fermat pseudoprime to base `2`;
///   (c) the Carmichael number `561` (carmichael_properties.ac, restated):
///       `561` is composite and `a^560 ≡ 1 (mod 561)` for every base `a`
///       coprime to `561`;
///   (d) the Miller-Rabin test — the strengthened Fermat test — is described
///       in a comment at the end of the file.

from nat import Nat
from nat import exp_mul, exp_one, one_exp, lt_add_suc, lt_trans, mul_comm,
    mul_one_left, small_mod, read_mul_single, read_read_carry, add_imp_sub
from number_theory.congruence import congr_mod_refl, congr_mod_symm,
    congr_mod_trans, congr_mod_mul, congr_mod_pow, mod_add_mul
from number_theory.fermat import fermat_euler
from number_theory.totient import coprime_below_prime
from number_theory.factorisation import coprime_of_distinct_primes
from number_theory.crt import nat_congr_combine_coprime
from number_theory.carmichael import carmichael_witness, carmichael_witness_member
from number_theory.carmichael_properties import eleven_is_prime,
    is_carmichael_number, is_carmichael_number_apply, is_carmichael_number_561,
    carmichael_witness_561, composite_561
from number_theory.mersenne_perfect import thirty_one_is_prime
numerals Nat

// ---------------------------------------------------------------------------
// The Fermat test.
// ---------------------------------------------------------------------------

/// The Fermat test: for a prime `n` and a base `a` coprime to `n`, the power
/// `a^(n - 1)` is congruent to `1` modulo `n`. This is `fermat_euler` from
/// fermat.ac, restated.
theorem fermat_test(n: Nat, a: Nat) {
    n.is_prime and a.coprime(n) implies a.pow(n - Nat.1).congr_mod(Nat.1, n)
} by {
    if n.is_prime and a.coprime(n) {
        fermat_euler(n, a)
        a.pow(n - Nat.1).congr_mod(Nat.1, n)
    }
}

/// The Fermat test with the coprimality hypothesis written as a unit gcd:
/// for a prime `n` and a base `a` with `gcd(a, n) = 1`, the power
/// `a^(n - 1)` is congruent to `1` modulo `n`. Coprimality with `n` is
/// exactly `gcd(a, n) = 1`.
theorem fermat_test_gcd(n: Nat, a: Nat) {
    n.is_prime and a.gcd(n) = Nat.1 implies a.pow(n - Nat.1).congr_mod(Nat.1, n)
} by {
    if n.is_prime and a.gcd(n) = Nat.1 {
        a.coprime(n)
        fermat_euler(n, a)
        a.pow(n - Nat.1).congr_mod(Nat.1, n)
    }
}

// ---------------------------------------------------------------------------
// 341 = 11 * 31 is a base-two Fermat pseudoprime.
// ---------------------------------------------------------------------------

/// `11 < 31`.
theorem eleven_lt_thirty_one {
    Nat.11 < Nat.31
} by {
    lt_add_suc(Nat.11, Nat.19)
    Nat.11 < Nat.11 + Nat.20
    Nat.11 + Nat.20 = Nat.31
    Nat.11 < Nat.31
}

/// `2 < 31`.
theorem two_lt_thirty_one {
    Nat.2 < Nat.31
} by {
    lt_trans(Nat.2, Nat.11, Nat.31)
    Nat.2 < Nat.11
    eleven_lt_thirty_one
    Nat.11 < Nat.31
    Nat.2 < Nat.31
}

/// The base `2` is coprime to the prime `11`.
theorem two_coprime_eleven {
    Nat.2.coprime(Nat.11)
} by {
    coprime_below_prime(Nat.11, Nat.2)
    eleven_is_prime
    Nat.11.is_prime
    Nat.1 <= Nat.2
    Nat.2 < Nat.11
    Nat.2.coprime(Nat.11)
}

/// The base `2` is coprime to the prime `31`.
theorem two_coprime_thirty_one {
    Nat.2.coprime(Nat.31)
} by {
    coprime_below_prime(Nat.31, Nat.2)
    thirty_one_is_prime
    Nat.31.is_prime
    Nat.1 <= Nat.2
    two_lt_thirty_one
    Nat.2 < Nat.31
    Nat.2.coprime(Nat.31)
}

/// The primes `11` and `31` are coprime.
theorem eleven_coprime_thirty_one {
    Nat.11.coprime(Nat.31)
} by {
    coprime_of_distinct_primes(Nat.11, Nat.31)
    eleven_is_prime
    Nat.11.is_prime
    thirty_one_is_prime
    Nat.31.is_prime
    Nat.11 != Nat.31
    Nat.11.coprime(Nat.31)
}

/// `341 = 11 * 31`.
theorem nat_mul_11_31 {
    Nat.11 * Nat.31 = Nat.341
} by {
    mul_comm(Nat.11, Nat.31)
    Nat.11 * Nat.31 = Nat.31 * Nat.11
    Nat.31 = Nat.3.read(Nat.1)
    read_mul_single(Nat.3, Nat.1, Nat.11)
    Nat.3.read(Nat.1) * Nat.11 = (Nat.3 * Nat.11).read(Nat.1 * Nat.11)
    Nat.3 * Nat.11 = Nat.33
    Nat.1 * Nat.11 = Nat.11
    Nat.3.read(Nat.1) * Nat.11 = Nat.33.read(Nat.11)
    Nat.33.read(Nat.11) = Nat.33.read(Nat.10 * Nat.1 + Nat.1)
    read_read_carry(Nat.33, Nat.1, Nat.1)
    Nat.33.read(Nat.10 * Nat.1 + Nat.1) = (Nat.33 + Nat.1).read(Nat.1)
    Nat.33 + Nat.1 = Nat.34
    Nat.33.read(Nat.11) = Nat.34.read(Nat.1)
    Nat.34.read(Nat.1) = Nat.341
    Nat.31 * Nat.11 = Nat.341
    Nat.11 * Nat.31 = Nat.341
}

/// `2^10 ≡ 1 (mod 11)`: Fermat's little theorem at the prime `11`.
theorem two_pow_ten_congr_one_mod_eleven {
    Nat.2.pow(Nat.10).congr_mod(Nat.1, Nat.11)
} by {
    eleven_is_prime
    Nat.11.is_prime
    two_coprime_eleven
    Nat.2.coprime(Nat.11)
    fermat_euler(Nat.11, Nat.2)
    Nat.2.pow(Nat.11 - Nat.1).congr_mod(Nat.1, Nat.11)
    Nat.11 - Nat.1 = Nat.10
    Nat.2.pow(Nat.10).congr_mod(Nat.1, Nat.11)
}

/// `2^30 ≡ 1 (mod 31)`: Fermat's little theorem at the prime `31`.
theorem two_pow_thirty_congr_one_mod_thirty_one {
    Nat.2.pow(Nat.30).congr_mod(Nat.1, Nat.31)
} by {
    thirty_one_is_prime
    Nat.31.is_prime
    two_coprime_thirty_one
    Nat.2.coprime(Nat.31)
    fermat_euler(Nat.31, Nat.2)
    Nat.2.pow(Nat.31 - Nat.1).congr_mod(Nat.1, Nat.31)
    Nat.31 - Nat.1 = Nat.30
    Nat.2.pow(Nat.30).congr_mod(Nat.1, Nat.31)
}

/// `2^5 ≡ 1 (mod 31)`: `2^5 = 32 = 31 + 1`.
theorem two_pow_five_congr_one_mod_thirty_one {
    Nat.2.pow(Nat.5).congr_mod(Nat.1, Nat.31)
} by {
    Nat.2.pow(Nat.5) = Nat.32
    mod_add_mul(Nat.1, Nat.31, Nat.1)
    (Nat.1 * Nat.31 + Nat.1).mod(Nat.31) = Nat.1.mod(Nat.31)
    Nat.1 * Nat.31 + Nat.1 = Nat.32
    Nat.32.mod(Nat.31) = Nat.1.mod(Nat.31)
    Nat.2.pow(Nat.5).mod(Nat.31) = Nat.1.mod(Nat.31)
    Nat.2.pow(Nat.5).congr_mod(Nat.1, Nat.31)
}

/// `2^10 ≡ 1 (mod 31)`: squaring `2^5 ≡ 1 (mod 31)`.
theorem two_pow_ten_congr_one_mod_thirty_one {
    Nat.2.pow(Nat.10).congr_mod(Nat.1, Nat.31)
} by {
    two_pow_five_congr_one_mod_thirty_one
    Nat.2.pow(Nat.5).congr_mod(Nat.1, Nat.31)
    congr_mod_pow(Nat.2.pow(Nat.5), Nat.1, Nat.31, Nat.2)
    Nat.2.pow(Nat.5).pow(Nat.2).congr_mod(Nat.1.pow(Nat.2), Nat.31)
    exp_mul(Nat.2, Nat.5, Nat.2)
    Nat.2.pow(Nat.5 * Nat.2) = Nat.2.pow(Nat.5).pow(Nat.2)
    Nat.5 * Nat.2 = Nat.10
    Nat.2.pow(Nat.10) = Nat.2.pow(Nat.5).pow(Nat.2)
    Nat.2.pow(Nat.10).congr_mod(Nat.1.pow(Nat.2), Nat.31)
    one_exp(Nat.2)
    Nat.1.pow(Nat.2) = Nat.1
    Nat.2.pow(Nat.10).congr_mod(Nat.1, Nat.31)
}

/// `2^340 ≡ 1 (mod 11)`: lift `2^10 ≡ 1 (mod 11)` along `340 = 10 * 34`.
theorem two_pow_340_congr_one_mod_eleven {
    Nat.2.pow(Nat.340).congr_mod(Nat.1, Nat.11)
} by {
    two_pow_ten_congr_one_mod_eleven
    Nat.2.pow(Nat.10).congr_mod(Nat.1, Nat.11)
    congr_mod_pow(Nat.2.pow(Nat.10), Nat.1, Nat.11, Nat.34)
    Nat.2.pow(Nat.10).pow(Nat.34).congr_mod(Nat.1.pow(Nat.34), Nat.11)
    exp_mul(Nat.2, Nat.10, Nat.34)
    Nat.2.pow(Nat.10 * Nat.34) = Nat.2.pow(Nat.10).pow(Nat.34)
    Nat.10 * Nat.34 = Nat.340
    Nat.2.pow(Nat.340) = Nat.2.pow(Nat.10).pow(Nat.34)
    Nat.2.pow(Nat.340).congr_mod(Nat.1.pow(Nat.34), Nat.11)
    one_exp(Nat.34)
    Nat.1.pow(Nat.34) = Nat.1
    Nat.2.pow(Nat.340).congr_mod(Nat.1, Nat.11)
}

/// `2^340 ≡ 1 (mod 31)`: lift `2^10 ≡ 1 (mod 31)` along `340 = 10 * 34`.
theorem two_pow_340_congr_one_mod_thirty_one {
    Nat.2.pow(Nat.340).congr_mod(Nat.1, Nat.31)
} by {
    two_pow_ten_congr_one_mod_thirty_one
    Nat.2.pow(Nat.10).congr_mod(Nat.1, Nat.31)
    congr_mod_pow(Nat.2.pow(Nat.10), Nat.1, Nat.31, Nat.34)
    Nat.2.pow(Nat.10).pow(Nat.34).congr_mod(Nat.1.pow(Nat.34), Nat.31)
    exp_mul(Nat.2, Nat.10, Nat.34)
    Nat.2.pow(Nat.10 * Nat.34) = Nat.2.pow(Nat.10).pow(Nat.34)
    Nat.10 * Nat.34 = Nat.340
    Nat.2.pow(Nat.340) = Nat.2.pow(Nat.10).pow(Nat.34)
    Nat.2.pow(Nat.340).congr_mod(Nat.1.pow(Nat.34), Nat.31)
    one_exp(Nat.34)
    Nat.1.pow(Nat.34) = Nat.1
    Nat.2.pow(Nat.340).congr_mod(Nat.1, Nat.31)
}

/// `2^340 ≡ 1 (mod 341)`: the CRT-based check. The congruence holds modulo
/// each of the coprime factors `11` and `31` of `341`, so it holds modulo
/// their product `341`.
theorem two_pow_340_congr_one_mod_341 {
    Nat.2.pow(Nat.340).congr_mod(Nat.1, Nat.341)
} by {
    two_pow_340_congr_one_mod_eleven
    Nat.2.pow(Nat.340).congr_mod(Nat.1, Nat.11)
    two_pow_340_congr_one_mod_thirty_one
    Nat.2.pow(Nat.340).congr_mod(Nat.1, Nat.31)
    nat_congr_combine_coprime(Nat.11, Nat.31, Nat.2.pow(Nat.340), Nat.1)
    eleven_coprime_thirty_one
    Nat.11.coprime(Nat.31)
    Nat.2.pow(Nat.340).congr_mod(Nat.1, Nat.11 * Nat.31)
    nat_mul_11_31
    Nat.11 * Nat.31 = Nat.341
    Nat.2.pow(Nat.340).congr_mod(Nat.1, Nat.341)
}

/// `341` is composite: `341 = 11 * 31` with both factors above one.
theorem composite_341 {
    Nat.341.is_composite
} by {
    Nat.1 < Nat.11
    Nat.1 < Nat.31
    nat_mul_11_31
    Nat.11 * Nat.31 = Nat.341
    exists(c: Nat) { Nat.1 < Nat.11 and Nat.1 < c and Nat.341 = Nat.11 * c }
    exists(b: Nat, c: Nat) {
        Nat.1 < b and Nat.1 < c and Nat.341 = b * c
    }
    Nat.341.is_composite = exists(b: Nat, c: Nat) {
        Nat.1 < b and Nat.1 < c and Nat.341 = b * c
    }
    Nat.341.is_composite
}

/// `341` is not prime.
theorem not_prime_341 {
    not Nat.341.is_prime
} by {
    if Nat.341.is_prime {
        Nat.341.is_prime = (Nat.1 < Nat.341 and not Nat.341.is_composite)
        Nat.1 < Nat.341 and not Nat.341.is_composite
        not Nat.341.is_composite
        composite_341
        Nat.341.is_composite
        false
    }
}

/// True when the composite `n` passes the Fermat test to base `a`: `n` is
/// composite and `a^(n - 1) ≡ 1 (mod n)`. Such an `n` is a Fermat
/// pseudoprime to base `a`.
define is_fermat_pseudoprime(a: Nat, n: Nat) -> Bool {
    n.is_composite and a.pow(n - Nat.1).congr_mod(Nat.1, n)
}

/// The pseudoprime predicate unfolds to compositeness plus the base-`a`
/// Fermat congruence.
theorem is_fermat_pseudoprime_apply(a: Nat, n: Nat) {
    is_fermat_pseudoprime(a, n) =
        (n.is_composite and a.pow(n - Nat.1).congr_mod(Nat.1, n))
}

/// `341` is a Fermat pseudoprime to base `2` — the first such number: `341`
/// is composite, yet `2^340 ≡ 1 (mod 341)`.
theorem fermat_pseudoprime_341 {
    is_fermat_pseudoprime(Nat.2, Nat.341)
} by {
    composite_341
    Nat.341.is_composite
    two_pow_340_congr_one_mod_341
    Nat.2.pow(Nat.340).congr_mod(Nat.1, Nat.341)
    Nat.341 - Nat.1 = Nat.340
    Nat.2.pow(Nat.341 - Nat.1).congr_mod(Nat.1, Nat.341)
    Nat.341.is_composite and Nat.2.pow(Nat.341 - Nat.1).congr_mod(Nat.1, Nat.341)
    is_fermat_pseudoprime_apply(Nat.2, Nat.341)
    is_fermat_pseudoprime(Nat.2, Nat.341) =
        (Nat.341.is_composite and Nat.2.pow(Nat.341 - Nat.1).congr_mod(Nat.1, Nat.341))
    is_fermat_pseudoprime(Nat.2, Nat.341)
}

/// The base-2 Fermat test is not sufficient for primality: the composite
/// number `341` passes it.
theorem base_two_fermat_test_insufficient {
    exists(n: Nat) {
        n.is_composite and Nat.2.pow(n - Nat.1).congr_mod(Nat.1, n)
    }
} by {
    composite_341
    Nat.341.is_composite
    two_pow_340_congr_one_mod_341
    Nat.2.pow(Nat.340).congr_mod(Nat.1, Nat.341)
    Nat.341 - Nat.1 = Nat.340
    Nat.2.pow(Nat.341 - Nat.1).congr_mod(Nat.1, Nat.341)
    Nat.341.is_composite and Nat.2.pow(Nat.341 - Nat.1).congr_mod(Nat.1, Nat.341)
    exists(n: Nat) {
        n.is_composite and Nat.2.pow(n - Nat.1).congr_mod(Nat.1, n)
    }
}

// ---------------------------------------------------------------------------
// 561 is a Carmichael number.
// ---------------------------------------------------------------------------

/// `561` is a Carmichael number: the composite number `561 = 3 * 11 * 17`
/// satisfies `a^560 ≡ 1 (mod 561)` for every base `a` coprime to `561`
/// (restated from `is_carmichael_number_561` in carmichael_properties.ac).
theorem carmichael_561 {
    is_carmichael_number(Nat.561)
} by {
    is_carmichael_number_561
    is_carmichael_number(Nat.561)
}

/// `561` is composite (restated from `composite_561` in
/// carmichael_properties.ac).
theorem composite_561_restated {
    Nat.561.is_composite
} by {
    composite_561
    Nat.561.is_composite
}

/// Every unit `a` modulo `561` satisfies `a^560 ≡ 1 (mod 561)`.
theorem carmichael_561_units(a: Nat) {
    a.coprime(Nat.561) implies a.pow(Nat.560).congr_mod(Nat.1, Nat.561)
} by {
    if a.coprime(Nat.561) {
        carmichael_witness_561
        carmichael_witness(Nat.561)(Nat.560)
        carmichael_witness_member(Nat.561, Nat.560, a)
        a.pow(Nat.560).congr_mod(Nat.1, Nat.561)
    }
}

/// `561` is composite and every unit `a` modulo `561` satisfies
/// `a^560 ≡ 1 (mod 561)`: a composite number passing the Fermat test for
/// every coprime base — the defining content of a Carmichael number.
theorem carmichael_561_composite_and_units {
    Nat.561.is_composite and forall(a: Nat) {
        a.coprime(Nat.561) implies a.pow(Nat.560).congr_mod(Nat.1, Nat.561)
    }
} by {
    composite_561_restated
    Nat.561.is_composite
    forall(a: Nat) {
        if a.coprime(Nat.561) {
            carmichael_561_units(a)
            a.pow(Nat.560).congr_mod(Nat.1, Nat.561)
        }
        a.coprime(Nat.561) implies a.pow(Nat.560).congr_mod(Nat.1, Nat.561)
    }
    Nat.561.is_composite and forall(a: Nat) {
        a.coprime(Nat.561) implies a.pow(Nat.560).congr_mod(Nat.1, Nat.561)
    }
}

/// `561` is not prime.
theorem not_prime_561 {
    not Nat.561.is_prime
} by {
    if Nat.561.is_prime {
        Nat.561.is_prime = (Nat.1 < Nat.561 and not Nat.561.is_composite)
        Nat.1 < Nat.561 and not Nat.561.is_composite
        not Nat.561.is_composite
        composite_561_restated
        Nat.561.is_composite
        false
    }
}

// ---------------------------------------------------------------------------
// The Miller-Rabin test (the strengthened Fermat test) — stated, not proved.
//
// The plain Fermat test checks `a^(n - 1) ≡ 1 (mod n)`; the pseudoprime 341
// above shows this is not sufficient for primality. The Miller-Rabin test
// strengthens it by factoring the exponent. Write `n - 1 = d * 2^s` with `d`
// odd. If `n` is prime, then for every base `a` coprime to `n`, either
// `a^d ≡ 1 (mod n)` or `a^(d * 2^r) ≡ -1 (mod n)` for some `0 <= r < s`
// (the sequence `a^d, a^(2d), ..., a^(2^(s-1) * d)` ends in `1` with each
// step squaring, so the step before the first `1` must be `-1`). A composite
// `n` for which some base `a` nevertheless satisfies the strengthened
// condition is a strong pseudoprime to base `a`; finding one base that
// fails it certifies `n` as composite.
//
// For `n = 341`: `n - 1 = 340 = 85 * 2^2`, so `d = 85` and `s = 2`. The
// base-2 Miller-Rabin test computes `2^85 (mod 341)`. Since `2^10 ≡ 1
// (mod 341)` (by CRT from the two congruences proved above),
// `2^85 = (2^10)^8 * 2^5 ≡ 2^5 = 32 (mod 341)`. As `32` is neither `1` nor
// `-1 = 340 (mod 341)`, the base-2 strengthened test rejects `341`, even
// though the plain Fermat test accepts it. Similarly the small composites
// 561, 1105 and 1729 are rejected by base-2 Miller-Rabin.
//
// Proving the Miller-Rabin theorem needs a congruence-friendly formalisation
// of `-1 (mod n)` as `n - 1` and of the iterated squaring chain, which is
// left for future work.
