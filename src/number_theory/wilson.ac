from list import product
from list import List, list_contains_implies_count_geq_one,
    unique_implies_no_duplicate, unique_implies_tail_unique
from list import map, map_contains, map_contains_of_contains
from list import is_permutation, permutation_preserves_product,
    product_remove_one, remove_one_cons_neq, remove_one_contains_other,
    remove_one_unique, remove_one_unique_not_contains_self,
    unique_same_contains_imp_permutation
from nat import Nat
from nat import add_cancels_left, add_imp_sub, add_mod,
    divides_factorial, divides_lte, divides_mod, divides_sub, divisor_lt,
    factorial_step, mod_by_zero, small_mod, strong_induction, sub_one_lt,
    suc_sub_one, true_below, add_one_right, lte_ref, lte_trans, lte_antisymm,
    lt_suc_right, not_lt_zero, lte_and_lt, lt_diff, mul_one_left, sub_lt,
    add_zero_left
from number_theory.congruence import congr_mod_mul, congr_mod_refl, congr_mod_symm,
    congr_mod_trans, mod_add_mul, mod_congr_mod_self, mod_lt
from number_theory.coprime import coprime_mod_imp, nat_divides_one_imp_one
from nat import gcd_zero_left
from number_theory.modular_inverse import mod_inv, mod_inv_coprime,
    mod_inv_mul_congr_one, mod_inv_unique_congr
from number_theory.fermat import prime_divides_mul
from number_theory.totient import coprime_below_prime, coprime_residues,
    coprime_residues_below, coprime_residues_below_suc_no,
    coprime_residues_below_suc_yes, coprime_residues_below_all_below,
    coprime_residues_below_all_coprime, coprime_residues_contains_imp,
    coprime_residues_contains_intro, coprime_residues_unique,
    congr_mod_below_eq, cons_unique_intro, product_coprime_residues_coprime

numerals Nat

/// A unique cons list has a head absent from its tail.
theorem unique_cons_not_tail_nat(head: Nat, tail: List[Nat]) {
    List.cons(head, tail).is_unique implies not tail.contains(head)
} by {
    if List.cons(head, tail).is_unique {
        if tail.contains(head) {
            list_contains_implies_count_geq_one(tail, head)
            tail.count(head) >= Nat.1
            List.cons(head, tail).count(head) = Nat.1 + tail.count(head)
            Nat.1 + Nat.1 <= Nat.1 + tail.count(head)
            Nat.1 + Nat.1 = Nat.2
            Nat.2 <= List.cons(head, tail).count(head)
            unique_implies_no_duplicate(List.cons(head, tail), head)
            List.cons(head, tail).count(head) <= Nat.1
            Nat.2 <= Nat.1
            false
        }
    }
}

/// Removing one occurrence from a natural-number list never increases length.
theorem remove_one_length_le_nat(list: List[Nat], item: Nat) {
    list.remove_one(item).length <= list.length
} by {
    define p(xs: List[Nat]) -> Bool {
        xs.remove_one(item).length <= xs.length
    }
    List.nil[Nat].remove_one(item) = List.nil[Nat]
    p(List.nil[Nat])
    forall(head: Nat, tail: List[Nat]) {
        if p(tail) {
            if head = item {
                List.cons(head, tail).remove_one(item) = tail
                tail.length < tail.length.suc
                List.cons(head, tail).length = tail.length.suc
                List.cons(head, tail).remove_one(item).length <= List.cons(head, tail).length
            } else {
                remove_one_cons_neq(head, tail, item)
                List.cons(head, tail).remove_one(item) = List.cons(head, tail.remove_one(item))
                tail.remove_one(item).length <= tail.length
                List.cons(head, tail.remove_one(item)).length = tail.remove_one(item).length.suc
                List.cons(head, tail).length = tail.length.suc
                tail.remove_one(item).length.suc <= tail.length.suc
                List.cons(head, tail).remove_one(item).length <= List.cons(head, tail).length
            }
            p(List.cons(head, tail))
        }
    }
    List.induction(function(xs: List[Nat]) { p(xs) })
    p(list)
}

/// Removing from a tail gives a list strictly shorter than the corresponding
/// cons list.
theorem remove_one_tail_length_lt_cons(head: Nat, tail: List[Nat], item: Nat) {
    tail.remove_one(item).length < List.cons(head, tail).length
} by {
    remove_one_length_le_nat(tail, item)
    tail.remove_one(item).length <= tail.length
    tail.length < tail.length.suc
    List.cons(head, tail).length = tail.length.suc
    tail.remove_one(item).length < List.cons(head, tail).length
}

/// In a unique list, an element remaining after `remove_one(item)` was already
/// in the original list.
theorem remove_one_unique_contains_imp_contains_nat(list: List[Nat], item: Nat, x: Nat) {
    list.is_unique and list.remove_one(item).contains(x) implies list.contains(x)
} by {
    if list.is_unique and list.remove_one(item).contains(x) {
        if x = item {
            remove_one_unique_not_contains_self(list, item)
            false
        } else {
            x != item
            remove_one_contains_other(list, item, x)
            list.contains(x) = list.remove_one(item).contains(x)
            list.contains(x)
        }
    }
}

/// In a unique list, the removed element is absent afterward.
theorem remove_one_unique_contains_imp_ne_item_nat(list: List[Nat], item: Nat, x: Nat) {
    list.is_unique and list.remove_one(item).contains(x) implies x != item
} by {
    if list.is_unique and list.remove_one(item).contains(x) {
        if x = item {
            remove_one_unique_not_contains_self(list, item)
            false
        }
    }
}

/// The induction predicate used to identify the product of prime reduced
/// residues below a bound with the preceding factorial.
define prime_coprime_residues_product_pred(p: Nat) -> (Nat -> Bool) {
    function(k: Nat) {
        k <= p implies
            product[Nat](coprime_residues_below(p, k)) = (k - Nat.1).factorial
    }
}

/// The modular inverse reduced into the canonical residue range.
define inv_mod_fn(n: Nat) -> (Nat -> Nat) {
    function(x: Nat) { mod_inv(x, n).mod(n) }
}

/// The reduced residues mapped through normalized modular inverse.
define inv_mod_residues(n: Nat) -> List[Nat] {
    map(coprime_residues(n), inv_mod_fn(n))
}

/// Multiplies a value by its normalized modular inverse.
define mul_inv_mod_fn(n: Nat) -> (Nat -> Nat) {
    function(x: Nat) { x * inv_mod_fn(n)(x) }
}

/// The reduced-residue list with the two fixed residues removed.
define wilson_nonfixed_residues(p: Nat) -> List[Nat] {
    coprime_residues(p).remove_one(Nat.1).remove_one(p - Nat.1)
}

/// True if every element of a list is a reduced residue modulo `p`.
define wilson_items_are_residues(p: Nat, items: List[Nat]) -> Bool {
    forall(x: Nat) { items.contains(x) implies coprime_residues(p).contains(x) }
}

/// True if a list excludes the two fixed residues `1` and `p - 1`.
define wilson_items_nonfixed(p: Nat, items: List[Nat]) -> Bool {
    forall(x: Nat) { items.contains(x) implies x != Nat.1 and x != p - Nat.1 }
}

/// True if a list is closed under normalized modular inversion.
define wilson_items_inv_closed(p: Nat, items: List[Nat]) -> Bool {
    forall(x: Nat) { items.contains(x) implies items.contains(inv_mod_fn(p)(x)) }
}

/// True if a list of residues can be paired by normalized modular inversion,
/// with the two fixed residues `1` and `p - 1` removed.
define wilson_pairable_list(p: Nat, items: List[Nat]) -> Bool {
    items.is_unique and wilson_items_are_residues(p, items) and
    wilson_items_nonfixed(p, items) and wilson_items_inv_closed(p, items)
}

/// A pairable list is unique.
theorem wilson_pairable_unique(p: Nat, items: List[Nat]) {
    wilson_pairable_list(p, items) implies items.is_unique
} by {
    if wilson_pairable_list(p, items) {
        items.is_unique and wilson_items_are_residues(p, items) and
        wilson_items_nonfixed(p, items) and wilson_items_inv_closed(p, items)
    }
}

/// Elements of a pairable list are reduced residues.
theorem wilson_pairable_contains_residue(p: Nat, items: List[Nat], x: Nat) {
    wilson_pairable_list(p, items) and items.contains(x)
        implies coprime_residues(p).contains(x)
} by {
    if wilson_pairable_list(p, items) and items.contains(x) {
        wilson_pairable_list(p, items) =
            (items.is_unique and wilson_items_are_residues(p, items) and
            wilson_items_nonfixed(p, items) and wilson_items_inv_closed(p, items))
        wilson_items_are_residues(p, items)
        wilson_items_are_residues(p, items) =
            forall(y: Nat) { items.contains(y) implies coprime_residues(p).contains(y) }
        forall(y: Nat) { items.contains(y) implies coprime_residues(p).contains(y) }
        coprime_residues(p).contains(x)
    }
}

/// Elements of a pairable list are not the two fixed residues.
theorem wilson_pairable_nonfixed(p: Nat, items: List[Nat], x: Nat) {
    wilson_pairable_list(p, items) and items.contains(x)
        implies x != Nat.1 and x != p - Nat.1
} by {
    if wilson_pairable_list(p, items) and items.contains(x) {
        wilson_pairable_list(p, items) =
            (items.is_unique and wilson_items_are_residues(p, items) and
            wilson_items_nonfixed(p, items) and wilson_items_inv_closed(p, items))
        wilson_items_nonfixed(p, items)
        wilson_items_nonfixed(p, items) =
            forall(y: Nat) { items.contains(y) implies y != Nat.1 and y != p - Nat.1 }
        forall(y: Nat) { items.contains(y) implies y != Nat.1 and y != p - Nat.1 }
        x != Nat.1 and x != p - Nat.1
    }
}

/// A pairable list is closed under normalized inversion.
theorem wilson_pairable_inv_mem(p: Nat, items: List[Nat], x: Nat) {
    wilson_pairable_list(p, items) and items.contains(x)
        implies items.contains(inv_mod_fn(p)(x))
} by {
    if wilson_pairable_list(p, items) and items.contains(x) {
        wilson_pairable_list(p, items) =
            (items.is_unique and wilson_items_are_residues(p, items) and
            wilson_items_nonfixed(p, items) and wilson_items_inv_closed(p, items))
        wilson_items_inv_closed(p, items)
        wilson_items_inv_closed(p, items) =
            forall(y: Nat) { items.contains(y) implies items.contains(inv_mod_fn(p)(y)) }
        forall(y: Nat) { items.contains(y) implies items.contains(inv_mod_fn(p)(y)) }
        items.contains(inv_mod_fn(p)(x))
    }
}

/// The normalized modular inverse is below a positive modulus.
theorem inv_mod_fn_lt(n: Nat, x: Nat) {
    n != Nat.0 implies inv_mod_fn(n)(x) < n
} by {
    if n != Nat.0 {
        inv_mod_fn(n)(x) = mod_inv(x, n).mod(n)
        mod_lt(mod_inv(x, n), n)
    }
}

/// The normalized modular inverse is coprime to the modulus whenever the
/// original value is.
theorem inv_mod_fn_coprime(n: Nat, x: Nat) {
    x.coprime(n) implies inv_mod_fn(n)(x).coprime(n)
} by {
    if x.coprime(n) {
        mod_inv_coprime(x, n)
        mod_inv(x, n).coprime(n)
        coprime_mod_imp(mod_inv(x, n), n)
        mod_inv(x, n).mod(n).coprime(n)
        inv_mod_fn(n)(x) = mod_inv(x, n).mod(n)
    }
}

/// The normalized modular inverse is congruent to the chosen modular inverse.
theorem inv_mod_fn_congr_mod_inv(n: Nat, x: Nat) {
    inv_mod_fn(n)(x).congr_mod(mod_inv(x, n), n)
} by {
    mod_congr_mod_self(mod_inv(x, n), n)
    mod_inv(x, n).mod(n).congr_mod(mod_inv(x, n), n)
    inv_mod_fn(n)(x) = mod_inv(x, n).mod(n)
}

/// The normalized modular inverse of a reduced residue is again a reduced
/// residue.
theorem inv_mod_fn_mem(n: Nat, x: Nat) {
    n != Nat.0 and coprime_residues(n).contains(x)
        implies coprime_residues(n).contains(inv_mod_fn(n)(x))
} by {
    if n != Nat.0 and coprime_residues(n).contains(x) {
        coprime_residues_contains_imp(n, x)
        x < n and x.coprime(n)
        x.coprime(n)
        inv_mod_fn_lt(n, x)
        inv_mod_fn(n)(x) < n
        inv_mod_fn_coprime(n, x)
        inv_mod_fn(n)(x).coprime(n)
        coprime_residues_contains_intro(n, inv_mod_fn(n)(x))
        coprime_residues(n).contains(inv_mod_fn(n)(x))
    }
}

/// The normalized modular inverse still multiplies to one modulo `n`.
theorem inv_mod_fn_mul_congr_one(n: Nat, x: Nat) {
    x.coprime(n) implies (x * inv_mod_fn(n)(x)).congr_mod(Nat.1, n)
} by {
    if x.coprime(n) {
        mod_inv_mul_congr_one(x, n)
        (x * mod_inv(x, n)).congr_mod(Nat.1, n)
        inv_mod_fn_congr_mod_inv(n, x)
        inv_mod_fn(n)(x).congr_mod(mod_inv(x, n), n)
        congr_mod_refl(x, n)
        x.congr_mod(x, n)
        congr_mod_mul(x, inv_mod_fn(n)(x), x, mod_inv(x, n), n)
        (x * inv_mod_fn(n)(x)).congr_mod(x * mod_inv(x, n), n)
        congr_mod_trans(x * inv_mod_fn(n)(x), x * mod_inv(x, n), Nat.1, n)
        (x * inv_mod_fn(n)(x)).congr_mod(Nat.1, n)
    }
}

/// Applying the normalized inverse twice is congruent to the original reduced
/// residue.
theorem inv_mod_fn_involutive_congr_on_residues(n: Nat, x: Nat) {
    n != Nat.0 and coprime_residues(n).contains(x)
        implies inv_mod_fn(n)(inv_mod_fn(n)(x)).congr_mod(x, n)
} by {
    if n != Nat.0 and coprime_residues(n).contains(x) {
        let y: Nat = inv_mod_fn(n)(x)
        let z: Nat = inv_mod_fn(n)(y)
        coprime_residues_contains_imp(n, x)
        x.coprime(n)
        x.coprime(n)
        inv_mod_fn_mem(n, x)
        coprime_residues(n).contains(y)
        coprime_residues_contains_imp(n, y)
        y.coprime(n)
        y.coprime(n)
        inv_mod_fn_mul_congr_one(n, x)
        (x * y).congr_mod(Nat.1, n)
        x * y = y * x
        (y * x).congr_mod(Nat.1, n)
        inv_mod_fn_mul_congr_one(n, y)
        (y * z).congr_mod(Nat.1, n)
        mod_inv_unique_congr(y, n, z, x)
        z.congr_mod(x, n)
        inv_mod_fn(n)(inv_mod_fn(n)(x)).congr_mod(x, n)
    }
}

/// Applying the normalized inverse twice stays inside the reduced residues.
theorem inv_mod_fn_double_mem(n: Nat, x: Nat) {
    n != Nat.0 and coprime_residues(n).contains(x)
        implies coprime_residues(n).contains(inv_mod_fn(n)(inv_mod_fn(n)(x)))
} by {
    if n != Nat.0 and coprime_residues(n).contains(x) {
        let y: Nat = inv_mod_fn(n)(x)
        inv_mod_fn_mem(n, x)
        coprime_residues(n).contains(y)
        inv_mod_fn_mem(n, y)
        coprime_residues(n).contains(inv_mod_fn(n)(y))
        inv_mod_fn(n)(y) = inv_mod_fn(n)(inv_mod_fn(n)(x))
        coprime_residues(n).contains(inv_mod_fn(n)(inv_mod_fn(n)(x)))
    }
}

/// Applying the normalized inverse twice gives a value below a positive
/// modulus.
theorem inv_mod_fn_double_lt(n: Nat, x: Nat) {
    n != Nat.0 and coprime_residues(n).contains(x)
        implies inv_mod_fn(n)(inv_mod_fn(n)(x)) < n
} by {
    if n != Nat.0 and coprime_residues(n).contains(x) {
        inv_mod_fn_double_mem(n, x)
        coprime_residues(n).contains(inv_mod_fn(n)(inv_mod_fn(n)(x)))
        coprime_residues_contains_imp(n, inv_mod_fn(n)(inv_mod_fn(n)(x)))
        inv_mod_fn(n)(inv_mod_fn(n)(x)) < n and
            inv_mod_fn(n)(inv_mod_fn(n)(x)).coprime(n)
        inv_mod_fn(n)(inv_mod_fn(n)(x)) < n
    }
}

/// The normalized modular inverse is an involution on the reduced residues.
theorem inv_mod_fn_involutive_on_residues(n: Nat, x: Nat) {
    n != Nat.0 and coprime_residues(n).contains(x)
        implies inv_mod_fn(n)(inv_mod_fn(n)(x)) = x
} by {
    if n != Nat.0 and coprime_residues(n).contains(x) {
        coprime_residues_contains_imp(n, x)
        x < n and x.coprime(n)
        x < n
        inv_mod_fn_double_lt(n, x)
        inv_mod_fn(n)(inv_mod_fn(n)(x)) < n
        inv_mod_fn_involutive_congr_on_residues(n, x)
        inv_mod_fn(n)(inv_mod_fn(n)(x)).congr_mod(x, n)
        congr_mod_below_eq(n, inv_mod_fn(n)(inv_mod_fn(n)(x)), x)
        inv_mod_fn(n)(inv_mod_fn(n)(x)) = x
    }
}

/// Every value in the normalized inverse image of the reduced residues is a
/// reduced residue.
theorem inv_mod_residues_in_coprime(n: Nat, y: Nat) {
    n != Nat.0 and inv_mod_residues(n).contains(y)
        implies coprime_residues(n).contains(y)
} by {
    if n != Nat.0 and inv_mod_residues(n).contains(y) {
        inv_mod_residues(n) = map(coprime_residues(n), inv_mod_fn(n))
        map(coprime_residues(n), inv_mod_fn(n)).contains(y)
        map_contains(coprime_residues(n), inv_mod_fn(n), y)
        let x: Nat satisfy {
            coprime_residues(n).contains(x) and inv_mod_fn(n)(x) = y
        }
        coprime_residues(n).contains(x)
        inv_mod_fn_mem(n, x)
        coprime_residues(n).contains(inv_mod_fn(n)(x))
        coprime_residues(n).contains(y)
    }
}

/// Every reduced residue appears in the normalized inverse image of the
/// reduced residues.
theorem coprime_in_inv_mod_residues(n: Nat, y: Nat) {
    n != Nat.0 and coprime_residues(n).contains(y)
        implies inv_mod_residues(n).contains(y)
} by {
    if n != Nat.0 and coprime_residues(n).contains(y) {
        let x: Nat = inv_mod_fn(n)(y)
        inv_mod_fn_mem(n, y)
        coprime_residues(n).contains(x)
        map_contains_of_contains(coprime_residues(n), inv_mod_fn(n), x)
        map(coprime_residues(n), inv_mod_fn(n)).contains(inv_mod_fn(n)(x))
        inv_mod_fn_involutive_on_residues(n, y)
        inv_mod_fn(n)(x) = y
        map(coprime_residues(n), inv_mod_fn(n)).contains(y)
        inv_mod_residues(n) = map(coprime_residues(n), inv_mod_fn(n))
        inv_mod_residues(n).contains(y)
    }
}

/// Inductive predicate for uniqueness of normalized inverse images below a
/// bound.
define inv_mod_residues_below_unique_pred(n: Nat) -> (Nat -> Bool) {
    function(k: Nat) {
        k <= n implies map(coprime_residues_below(n, k), inv_mod_fn(n)).is_unique
    }
}

/// Step for uniqueness of normalized inverse images below a bound.
theorem inv_mod_residues_below_unique_step(n: Nat, k: Nat) {
    n != Nat.0 and inv_mod_residues_below_unique_pred(n)(k)
        implies inv_mod_residues_below_unique_pred(n)(k.suc)
} by {
    if n != Nat.0 and inv_mod_residues_below_unique_pred(n)(k) {
        if k.suc <= n {
            k <= n
            k < n
            map(coprime_residues_below(n, k), inv_mod_fn(n)).is_unique
            if k.coprime(n) {
                coprime_residues_below_suc_yes(n, k)
                coprime_residues_below(n, k.suc) =
                    List.cons(k, coprime_residues_below(n, k))
                map(coprime_residues_below(n, k.suc), inv_mod_fn(n)) =
                    List.cons(inv_mod_fn(n)(k),
                        map(coprime_residues_below(n, k), inv_mod_fn(n)))
                let mapped: List[Nat] = map(coprime_residues_below(n, k), inv_mod_fn(n))
                if mapped.contains(inv_mod_fn(n)(k)) {
                    map_contains(coprime_residues_below(n, k), inv_mod_fn(n), inv_mod_fn(n)(k))
                    let x: Nat satisfy {
                        coprime_residues_below(n, k).contains(x) and
                            inv_mod_fn(n)(x) = inv_mod_fn(n)(k)
                    }
                    coprime_residues_below(n, k).contains(x)
                    coprime_residues_below_all_below(n, k, x)
                    x < k
                    x < n
                    coprime_residues_below_all_coprime(n, k, x)
                    x.coprime(n)
                    coprime_residues_contains_intro(n, x)
                    coprime_residues(n).contains(x)
                    coprime_residues_contains_intro(n, k)
                    coprime_residues(n).contains(k)
                    inv_mod_fn_involutive_on_residues(n, x)
                    inv_mod_fn(n)(inv_mod_fn(n)(x)) = x
                    inv_mod_fn_involutive_on_residues(n, k)
                    inv_mod_fn(n)(inv_mod_fn(n)(k)) = k
                    inv_mod_fn(n)(x) = inv_mod_fn(n)(k)
                    inv_mod_fn(n)(inv_mod_fn(n)(x)) = inv_mod_fn(n)(inv_mod_fn(n)(k))
                    x = k
                    x < x
                    false
                }
                not mapped.contains(inv_mod_fn(n)(k))
                cons_unique_intro(inv_mod_fn(n)(k), mapped)
                List.cons(inv_mod_fn(n)(k), mapped).is_unique
                map(coprime_residues_below(n, k.suc), inv_mod_fn(n)).is_unique
            } else {
                coprime_residues_below_suc_no(n, k)
                coprime_residues_below(n, k.suc) = coprime_residues_below(n, k)
                map(coprime_residues_below(n, k.suc), inv_mod_fn(n)) =
                    map(coprime_residues_below(n, k), inv_mod_fn(n))
                map(coprime_residues_below(n, k.suc), inv_mod_fn(n)).is_unique
            }
        }
    }
}

/// Inner induction for uniqueness of normalized inverse images below a bound.
theorem inv_mod_residues_below_unique_run(n: Nat, k: Nat) {
    n != Nat.0 implies inv_mod_residues_below_unique_pred(n)(k)
} by {
    if n != Nat.0 {
        let f: Nat -> Bool = function(m: Nat) {
            m <= n implies map(coprime_residues_below(n, m), inv_mod_fn(n)).is_unique
        }
        forall(m: Nat) {
            inv_mod_residues_below_unique_pred(n)(m) = f(m)
            f(m) = inv_mod_residues_below_unique_pred(n)(m)
        }
        coprime_residues_below(n, Nat.0) = List.nil[Nat]
        map(List.nil[Nat], inv_mod_fn(n)) = List.nil[Nat]
        List.nil[Nat].is_unique
        f(Nat.0)
        forall(m: Nat) {
            if f(m) {
                inv_mod_residues_below_unique_pred(n)(m)
                inv_mod_residues_below_unique_step(n, m)
                inv_mod_residues_below_unique_pred(n)(m.suc)
                f(m.suc)
            }
        }
        f(k)
        inv_mod_residues_below_unique_pred(n)(k)
    }
}

/// Normalized inverse images below `k <= n` are unique.
theorem inv_mod_residues_below_unique(n: Nat, k: Nat) {
    n != Nat.0 and k <= n
        implies map(coprime_residues_below(n, k), inv_mod_fn(n)).is_unique
} by {
    if n != Nat.0 and k <= n {
        inv_mod_residues_below_unique_run(n, k)
        inv_mod_residues_below_unique_pred(n)(k)
    }
}

/// The normalized inverse image of the full reduced-residue list is unique.
theorem inv_mod_residues_unique(n: Nat) {
    n != Nat.0 implies inv_mod_residues(n).is_unique
} by {
    if n != Nat.0 {
        n <= n
        inv_mod_residues_below_unique(n, n)
        map(coprime_residues_below(n, n), inv_mod_fn(n)).is_unique
        coprime_residues(n) = coprime_residues_below(n, n)
        inv_mod_residues(n) = map(coprime_residues(n), inv_mod_fn(n))
        inv_mod_residues(n).is_unique
    }
}

/// The normalized inverse image and the reduced-residue list have the same
/// membership predicate.
theorem inv_mod_residues_same_contains(n: Nat) {
    n != Nat.0 implies forall(y: Nat) {
        inv_mod_residues(n).contains(y) = coprime_residues(n).contains(y)
    }
} by {
    if n != Nat.0 {
        forall(y: Nat) {
            if inv_mod_residues(n).contains(y) {
                inv_mod_residues_in_coprime(n, y)
                coprime_residues(n).contains(y)
            }
            if coprime_residues(n).contains(y) {
                coprime_in_inv_mod_residues(n, y)
                inv_mod_residues(n).contains(y)
            }
            inv_mod_residues(n).contains(y) = coprime_residues(n).contains(y)
        }
    }
}

/// The normalized inverse image of the reduced residues is a permutation of the
/// reduced residues.
theorem inv_mod_residues_is_permutation(n: Nat) {
    n != Nat.0 implies is_permutation(inv_mod_residues(n), coprime_residues(n))
} by {
    if n != Nat.0 {
        inv_mod_residues_unique(n)
        inv_mod_residues(n).is_unique
        coprime_residues_unique(n)
        coprime_residues(n).is_unique
        inv_mod_residues_same_contains(n)
        forall(y: Nat) {
            inv_mod_residues(n).contains(y) = coprime_residues(n).contains(y)
        }
        unique_same_contains_imp_permutation(inv_mod_residues(n), coprime_residues(n))
        is_permutation(inv_mod_residues(n), coprime_residues(n))
    }
}

/// The product of the normalized inverse image of the reduced residues equals
/// the product of the reduced residues.
theorem product_inv_mod_residues_eq_product(n: Nat) {
    n != Nat.0 implies
        product[Nat](inv_mod_residues(n)) = product[Nat](coprime_residues(n))
} by {
    if n != Nat.0 {
        inv_mod_residues_is_permutation(n)
        is_permutation(inv_mod_residues(n), coprime_residues(n))
        permutation_preserves_product(inv_mod_residues(n), coprime_residues(n))
    }
}

/// Predicate for products of `x * inv(x)` over a list.
define product_mul_inv_congr_one_pred(n: Nat) -> (List[Nat] -> Bool) {
    function(items: List[Nat]) {
        (forall(x: Nat) { items.contains(x) implies x.coprime(n) }) implies
            product[Nat](map(items, mul_inv_mod_fn(n))).congr_mod(Nat.1, n)
    }
}

/// Empty product for `x * inv(x)` is congruent to one.
theorem product_mul_inv_congr_one_nil(n: Nat) {
    product_mul_inv_congr_one_pred(n)(List.nil[Nat])
} by {
    if forall(x: Nat) { List.nil[Nat].contains(x) implies x.coprime(n) } {
        map(List.nil[Nat], mul_inv_mod_fn(n)) = List.nil[Nat]
        product[Nat](List.nil[Nat]) = Nat.1
        congr_mod_refl(Nat.1, n)
        product[Nat](map(List.nil[Nat], mul_inv_mod_fn(n))).congr_mod(Nat.1, n)
    }
}

/// Cons step for products of `x * inv(x)`.
theorem product_mul_inv_congr_one_cons(n: Nat, head: Nat, tail: List[Nat]) {
    product_mul_inv_congr_one_pred(n)(tail)
        implies product_mul_inv_congr_one_pred(n)(List.cons(head, tail))
} by {
    if product_mul_inv_congr_one_pred(n)(tail) {
        if forall(x: Nat) { List.cons(head, tail).contains(x) implies x.coprime(n) } {
            List.cons(head, tail).contains(head)
            head.coprime(n)
            forall(x: Nat) {
                if tail.contains(x) {
                    List.cons(head, tail).contains(x)
                    x.coprime(n)
                }
            }
            product[Nat](map(tail, mul_inv_mod_fn(n))).congr_mod(Nat.1, n)
            inv_mod_fn_mul_congr_one(n, head)
            (head * inv_mod_fn(n)(head)).congr_mod(Nat.1, n)
            mul_inv_mod_fn(n)(head) = head * inv_mod_fn(n)(head)
            mul_inv_mod_fn(n)(head).congr_mod(Nat.1, n)
            congr_mod_mul(
                mul_inv_mod_fn(n)(head), product[Nat](map(tail, mul_inv_mod_fn(n))),
                Nat.1, Nat.1, n)
            let head_tail_product: Nat =
                mul_inv_mod_fn(n)(head) * product[Nat](map(tail, mul_inv_mod_fn(n)))
            head_tail_product.congr_mod(Nat.1 * Nat.1, n)
            Nat.1 * Nat.1 = Nat.1
            map(List.cons(head, tail), mul_inv_mod_fn(n)) =
                List.cons(mul_inv_mod_fn(n)(head), map(tail, mul_inv_mod_fn(n)))
            product[Nat](map(List.cons(head, tail), mul_inv_mod_fn(n))) =
                mul_inv_mod_fn(n)(head) * product[Nat](map(tail, mul_inv_mod_fn(n)))
            product[Nat](map(List.cons(head, tail), mul_inv_mod_fn(n))).congr_mod(Nat.1, n)
        }
    }
}

/// The product of `x * inv(x)` over any list of values coprime to `n` is
/// congruent to one modulo `n`.
theorem product_mul_inv_congr_one(n: Nat, items: List[Nat]) {
    (forall(x: Nat) { items.contains(x) implies x.coprime(n) }) implies
        product[Nat](map(items, mul_inv_mod_fn(n))).congr_mod(Nat.1, n)
} by {
    define p(xs: List[Nat]) -> Bool {
        product_mul_inv_congr_one_pred(n)(xs)
    }
    product_mul_inv_congr_one_nil(n)
    p(List.nil[Nat])
    forall(head: Nat, tail: List[Nat]) {
        if p(tail) {
            product_mul_inv_congr_one_cons(n, head, tail)
            p(List.cons(head, tail))
        }
    }
    List.induction(function(xs: List[Nat]) { p(xs) })
    p(items)
    product_mul_inv_congr_one_pred(n)(items)
}

/// Specialization of the `x * inv(x)` product congruence to reduced residues.
theorem product_coprime_residues_mul_inv_congr_one(n: Nat) {
    product[Nat](map(coprime_residues(n), mul_inv_mod_fn(n))).congr_mod(Nat.1, n)
} by {
    forall(x: Nat) {
        if coprime_residues(n).contains(x) {
            coprime_residues_contains_imp(n, x)
            x < n and x.coprime(n)
            x.coprime(n)
        }
    }
    product_mul_inv_congr_one(n, coprime_residues(n))
}

/// Predicate for distributing the product of pointwise `x * inv(x)`.
define product_mul_inv_distrib_pred(n: Nat) -> (List[Nat] -> Bool) {
    function(items: List[Nat]) {
        product[Nat](map(items, mul_inv_mod_fn(n))) =
            product[Nat](items) * product[Nat](map(items, inv_mod_fn(n)))
    }
}

/// Empty-list case for distributing the product of pointwise `x * inv(x)`.
theorem product_mul_inv_distrib_nil(n: Nat) {
    product_mul_inv_distrib_pred(n)(List.nil[Nat])
} by {
    map(List.nil[Nat], mul_inv_mod_fn(n)) = List.nil[Nat]
    map(List.nil[Nat], inv_mod_fn(n)) = List.nil[Nat]
    product[Nat](List.nil[Nat]) = Nat.1
    Nat.1 * Nat.1 = Nat.1
}

/// Cons step for distributing the product of pointwise `x * inv(x)`.
theorem product_mul_inv_distrib_cons(n: Nat, head: Nat, tail: List[Nat]) {
    product_mul_inv_distrib_pred(n)(tail)
        implies product_mul_inv_distrib_pred(n)(List.cons(head, tail))
} by {
    if product_mul_inv_distrib_pred(n)(tail) {
        product[Nat](map(tail, mul_inv_mod_fn(n))) =
            product[Nat](tail) * product[Nat](map(tail, inv_mod_fn(n)))
        map(List.cons(head, tail), mul_inv_mod_fn(n)) =
            List.cons(mul_inv_mod_fn(n)(head), map(tail, mul_inv_mod_fn(n)))
        map(List.cons(head, tail), inv_mod_fn(n)) =
            List.cons(inv_mod_fn(n)(head), map(tail, inv_mod_fn(n)))
        product[Nat](map(List.cons(head, tail), mul_inv_mod_fn(n))) =
            mul_inv_mod_fn(n)(head) * product[Nat](map(tail, mul_inv_mod_fn(n)))
        mul_inv_mod_fn(n)(head) = head * inv_mod_fn(n)(head)
        product[Nat](map(List.cons(head, tail), mul_inv_mod_fn(n))) =
            (head * inv_mod_fn(n)(head)) *
                (product[Nat](tail) * product[Nat](map(tail, inv_mod_fn(n))))
        let a: Nat = head
        let b: Nat = inv_mod_fn(n)(head)
        let c: Nat = product[Nat](tail)
        let d: Nat = product[Nat](map(tail, inv_mod_fn(n)))
        (a * b) * (c * d) = a * b * (c * d)
        a * b * (c * d) = a * (b * c) * d
        b * c = c * b
        a * (b * c) * d = a * (c * b) * d
        a * (c * b) * d = a * c * (b * d)
        a * c * (b * d) = (a * c) * (b * d)
        (a * b) * (c * d) = (a * c) * (b * d)
        (head * inv_mod_fn(n)(head)) *
                (product[Nat](tail) * product[Nat](map(tail, inv_mod_fn(n)))) =
            (head * product[Nat](tail)) *
                (inv_mod_fn(n)(head) * product[Nat](map(tail, inv_mod_fn(n))))
        product[Nat](List.cons(head, tail)) = head * product[Nat](tail)
        product[Nat](map(List.cons(head, tail), inv_mod_fn(n))) =
            inv_mod_fn(n)(head) * product[Nat](map(tail, inv_mod_fn(n)))
        product[Nat](map(List.cons(head, tail), mul_inv_mod_fn(n))) =
            product[Nat](List.cons(head, tail)) *
                product[Nat](map(List.cons(head, tail), inv_mod_fn(n)))
    }
}

/// The product of pointwise `x * inv(x)` distributes into the original product
/// times the inverse-image product.
theorem product_mul_inv_distrib(n: Nat, items: List[Nat]) {
    product[Nat](map(items, mul_inv_mod_fn(n))) =
        product[Nat](items) * product[Nat](map(items, inv_mod_fn(n)))
} by {
    define p(xs: List[Nat]) -> Bool {
        product_mul_inv_distrib_pred(n)(xs)
    }
    product_mul_inv_distrib_nil(n)
    p(List.nil[Nat])
    forall(head: Nat, tail: List[Nat]) {
        if p(tail) {
            product_mul_inv_distrib_cons(n, head, tail)
            p(List.cons(head, tail))
        }
    }
    List.induction(function(xs: List[Nat]) { p(xs) })
    p(items)
    product_mul_inv_distrib_pred(n)(items)
}

/// The product of all reduced residues is self-inverse modulo `n`.
theorem product_coprime_residues_square_congr_one(n: Nat) {
    n != Nat.0 implies
        (product[Nat](coprime_residues(n)) * product[Nat](coprime_residues(n))).congr_mod(Nat.1, n)
} by {
    if n != Nat.0 {
        let residues: List[Nat] = coprime_residues(n)
        let residue_product: Nat = product[Nat](residues)
        product_coprime_residues_mul_inv_congr_one(n)
        product[Nat](map(residues, mul_inv_mod_fn(n))).congr_mod(Nat.1, n)
        product_mul_inv_distrib(n, residues)
        product[Nat](map(residues, mul_inv_mod_fn(n))) =
            product[Nat](residues) * product[Nat](map(residues, inv_mod_fn(n)))
        inv_mod_residues(n) = map(residues, inv_mod_fn(n))
        product[Nat](map(residues, inv_mod_fn(n))) = product[Nat](inv_mod_residues(n))
        product_inv_mod_residues_eq_product(n)
        product[Nat](inv_mod_residues(n)) = residue_product
        product[Nat](map(residues, inv_mod_fn(n))) = residue_product
        product[Nat](map(residues, mul_inv_mod_fn(n))) = residue_product * residue_product
        (residue_product * residue_product).congr_mod(Nat.1, n)
        (product[Nat](coprime_residues(n)) * product[Nat](coprime_residues(n))).congr_mod(Nat.1, n)
    }
}

/// The zero case for the product of prime reduced residues.
theorem prime_coprime_residues_product_zero(p: Nat) {
    prime_coprime_residues_product_pred(p)(Nat.0)
} by {
    if Nat.0 <= p {
        coprime_residues_below(p, Nat.0) = List.nil[Nat]
        product[Nat](List.nil[Nat]) = Nat.1
        Nat.0 - Nat.1 = Nat.0
        Nat.0.factorial = Nat.1
        product[Nat](coprime_residues_below(p, Nat.0)) = (Nat.0 - Nat.1).factorial
    }
}

/// Zero is not coprime to a prime.
theorem zero_not_coprime_prime(p: Nat) {
    p.is_prime implies not Nat.0.coprime(p)
} by {
    if p.is_prime {
        gcd_zero_left(p)
        Nat.0.gcd(p) = p
        Nat.1 < p
        p != Nat.1
        if Nat.0.coprime(p) {
            Nat.0.gcd(p) = Nat.1
            p = Nat.1
            false
        }
    }
}

/// A positive natural is its predecessor plus one.
theorem sub_one_add_one_eq_self(n: Nat) {
    n != Nat.0 implies (n - Nat.1) + Nat.1 = n
} by {
    if n != Nat.0 {
        let pred: Nat satisfy { pred.suc = n }
        n = pred.suc
        pred.suc = pred + Nat.1
        n = pred + Nat.1
        n - Nat.1 = pred
        (n - Nat.1) + Nat.1 = pred + Nat.1
        (n - Nat.1) + Nat.1 = n
    }
}

/// A number strictly below a positive bound is at most the predecessor of that
/// bound.
theorem lt_imp_le_sub_one(a: Nat, n: Nat) {
    a < n implies a <= n - Nat.1
} by {
    if a < n {
        let d: Nat satisfy { a + d = n and d != Nat.0 }
        let e: Nat satisfy { e.suc = d }
        d = e.suc
        e.suc = e + Nat.1
        a + d = a + (e + Nat.1)
        a + (e + Nat.1) = a + e + Nat.1
        a + e + Nat.1 = n
        add_imp_sub(a + e, Nat.1, n)
        n - Nat.1 = a + e
        a <= a + e
        a <= n - Nat.1
    }
}

/// A positive natural minus its predecessor is one.
theorem sub_pred_eq_one(n: Nat) {
    n != Nat.0 implies n - (n - Nat.1) = Nat.1
} by {
    if n != Nat.0 {
        sub_one_add_one_eq_self(n)
        (n - Nat.1) + Nat.1 = n
        Nat.1 + (n - Nat.1) = n
        add_imp_sub(Nat.1, n - Nat.1, n)
        n - (n - Nat.1) = Nat.1
    }
}

/// If `1 < n`, then `1 <= n - 1`.
theorem one_le_sub_one_of_one_lt(n: Nat) {
    Nat.1 < n implies Nat.1 <= n - Nat.1
} by {
    if Nat.1 < n {
        let pred: Nat satisfy { pred.suc = n }
        n - Nat.1 = pred
        Nat.1 <= pred
        Nat.1 <= n - Nat.1
    }
}

/// Every natural greater than one is either two or greater than two.
theorem one_lt_eq_two_or_two_lt(n: Nat) {
    Nat.1 < n implies n = Nat.2 or Nat.2 < n
} by {
    if Nat.1 < n {
        if n = Nat.2 {
            n = Nat.2 or Nat.2 < n
        } else {
            Nat.2 < n
            n = Nat.2 or Nat.2 < n
        }
    }
}

/// Difference-of-squares at one, in a Nat-friendly form.
theorem pred_mul_suc_add_one_eq_square(x: Nat) {
    x != Nat.0 implies (x - Nat.1) * (x + Nat.1) + Nat.1 = x * x
} by {
    if x != Nat.0 {
        let pred: Nat satisfy { pred.suc = x }
        x = pred.suc
        x - Nat.1 = pred
        x + Nat.1 = pred.suc.suc
        (x - Nat.1) * (x + Nat.1) = pred * pred.suc.suc
        pred * pred.suc.suc = pred * pred.suc + pred
        pred.suc * pred.suc = pred * pred.suc + pred.suc
        pred.suc = pred + Nat.1
        pred * pred.suc + pred.suc = pred * pred.suc + pred + Nat.1
        pred * pred.suc + pred + Nat.1 = (pred * pred.suc + pred) + Nat.1
        pred.suc * pred.suc = pred * pred.suc + pred + Nat.1
        (x - Nat.1) * (x + Nat.1) + Nat.1 = pred * pred.suc + pred + Nat.1
        (x - Nat.1) * (x + Nat.1) + Nat.1 = pred.suc * pred.suc
        x * x = pred.suc * pred.suc
        (x - Nat.1) * (x + Nat.1) + Nat.1 = x * x
    }
}

/// A nonzero self-inverse modulo a prime makes `p` divide `(x - 1)(x + 1)`.
theorem prime_self_inverse_divides_pred_mul_suc(p: Nat, x: Nat) {
    p.is_prime and x != Nat.0 and (x * x).congr_mod(Nat.1, p)
        implies p.divides((x - Nat.1) * (x + Nat.1))
} by {
    if p.is_prime and x != Nat.0 and (x * x).congr_mod(Nat.1, p) {
        Nat.1 < p
        small_mod(Nat.1, p)
        Nat.1.mod(p) = Nat.1
        (x * x).mod(p) = Nat.1.mod(p)
        (x * x).mod(p) = Nat.1
        add_mod(x * x, p)
        let q: Nat satisfy { q * p + (x * x).mod(p) = x * x }
        q * p + Nat.1 = x * x
        pred_mul_suc_add_one_eq_square(x)
        (x - Nat.1) * (x + Nat.1) + Nat.1 = x * x
        q * p + Nat.1 = (x - Nat.1) * (x + Nat.1) + Nat.1
        Nat.1 + q * p = q * p + Nat.1
        Nat.1 + ((x - Nat.1) * (x + Nat.1)) = (x - Nat.1) * (x + Nat.1) + Nat.1
        Nat.1 + q * p = Nat.1 + ((x - Nat.1) * (x + Nat.1))
        add_cancels_left(Nat.1, q * p, (x - Nat.1) * (x + Nat.1))
        q * p = (x - Nat.1) * (x + Nat.1)
        p * q = q * p
        p.divides((x - Nat.1) * (x + Nat.1))
    }
}

/// Zero is not coprime to a prime.
theorem zero_not_coprime_prime_right(p: Nat) {
    p.is_prime implies not Nat.0.coprime(p)
} by {
    if p.is_prime {
        zero_not_coprime_prime(p)
    }
}

/// A reduced residue that is self-inverse modulo a prime is `1` or `p - 1`.
theorem prime_self_inverse_residue_eq_one_or_pred(p: Nat, x: Nat) {
    p.is_prime and x < p and x.coprime(p) and (x * x).congr_mod(Nat.1, p)
        implies x = Nat.1 or x = p - Nat.1
} by {
    if p.is_prime and x < p and x.coprime(p) and (x * x).congr_mod(Nat.1, p) {
        if x = Nat.0 {
            zero_not_coprime_prime_right(p)
            false
        }
        x != Nat.0
        prime_self_inverse_divides_pred_mul_suc(p, x)
        p.divides((x - Nat.1) * (x + Nat.1))
        prime_divides_mul(p, x - Nat.1, x + Nat.1)
        p.divides(x - Nat.1) or p.divides(x + Nat.1)
        if p.divides(x - Nat.1) {
            x - Nat.1 < p
            divides_lte(p, x - Nat.1)
            (x - Nat.1) = Nat.0 or p <= x - Nat.1
            if p <= x - Nat.1 {
                false
            }
            x - Nat.1 = Nat.0
            sub_one_add_one_eq_self(x)
            (x - Nat.1) + Nat.1 = x
            x = Nat.1
            x = Nat.1 or x = p - Nat.1
        } else {
            p.divides(x + Nat.1)
            x + Nat.1 <= p
            divides_lte(p, x + Nat.1)
            (x + Nat.1) = Nat.0 or p <= x + Nat.1
            if x + Nat.1 = Nat.0 {
                false
            }
            p <= x + Nat.1
            p = x + Nat.1
            add_imp_sub(x, Nat.1, p)
            p - Nat.1 = x
            x = p - Nat.1
            x = Nat.1 or x = p - Nat.1
        }
    }
}

/// A fixed point of normalized inversion has square congruent to one.
theorem inv_fixed_square_congr_one(n: Nat, x: Nat) {
    x.coprime(n) and inv_mod_fn(n)(x) = x implies (x * x).congr_mod(Nat.1, n)
} by {
    if x.coprime(n) and inv_mod_fn(n)(x) = x {
        inv_mod_fn_mul_congr_one(n, x)
        (x * inv_mod_fn(n)(x)).congr_mod(Nat.1, n)
        x * inv_mod_fn(n)(x) = x * x
        (x * x).congr_mod(Nat.1, n)
    }
}

/// A fixed point of normalized inversion among prime reduced residues is one
/// of the two trivial self-inverse residues.
theorem prime_inv_fixed_eq_one_or_pred(p: Nat, x: Nat) {
    p.is_prime and coprime_residues(p).contains(x) and inv_mod_fn(p)(x) = x
        implies x = Nat.1 or x = p - Nat.1
} by {
    if p.is_prime and coprime_residues(p).contains(x) and inv_mod_fn(p)(x) = x {
        coprime_residues_contains_imp(p, x)
        x < p and x.coprime(p)
        x < p
        x.coprime(p)
        inv_fixed_square_congr_one(p, x)
        (x * x).congr_mod(Nat.1, p)
        prime_self_inverse_residue_eq_one_or_pred(p, x)
        x = Nat.1 or x = p - Nat.1
    }
}

/// One is a reduced residue modulo a prime.
theorem prime_one_residue(p: Nat) {
    p.is_prime implies coprime_residues(p).contains(Nat.1)
} by {
    if p.is_prime {
        Nat.1 < p
        coprime_residues_contains_intro(p, Nat.1)
    }
}

/// The predecessor of a prime is a reduced residue modulo that prime.
theorem prime_pred_residue(p: Nat) {
    p.is_prime implies coprime_residues(p).contains(p - Nat.1)
} by {
    if p.is_prime {
        Nat.1 < p
        p != Nat.0
        sub_one_lt(p)
        p - Nat.1 < p
        one_le_sub_one_of_one_lt(p)
        Nat.1 <= p - Nat.1
        coprime_below_prime(p, p - Nat.1)
        (p - Nat.1).coprime(p)
        coprime_residues_contains_intro(p, p - Nat.1)
        coprime_residues(p).contains(p - Nat.1)
    }
}

/// For primes greater than two, the two fixed residues are distinct.
theorem prime_one_ne_pred_of_gt_two(p: Nat) {
    p.is_prime and Nat.2 < p implies Nat.1 != p - Nat.1
} by {
    if p.is_prime and Nat.2 < p {
        if Nat.1 = p - Nat.1 {
            sub_one_add_one_eq_self(p)
            (p - Nat.1) + Nat.1 = p
            Nat.1 + Nat.1 = p
            Nat.1 + Nat.1 = Nat.2
            p = Nat.2
            Nat.2 < Nat.2
            false
        }
    }
}

/// The predecessor of a prime is self-inverse modulo that prime.
theorem prime_pred_self_inverse_congr(p: Nat) {
    p.is_prime implies ((p - Nat.1) * (p - Nat.1)).congr_mod(Nat.1, p)
} by {
    if p.is_prime {
        Nat.1 < p
        p != Nat.0
        let x: Nat = p - Nat.1
        sub_one_add_one_eq_self(p)
        x + Nat.1 = p
        x != Nat.0
        pred_mul_suc_add_one_eq_square(x)
        (x - Nat.1) * (x + Nat.1) + Nat.1 = x * x
        (x - Nat.1) * p + Nat.1 = x * x
        mod_add_mul(x - Nat.1, p, Nat.1)
        ((x - Nat.1) * p + Nat.1).mod(p) = Nat.1.mod(p)
        (x * x).mod(p) = Nat.1.mod(p)
        (x * x).congr_mod(Nat.1, p)
        ((p - Nat.1) * (p - Nat.1)).congr_mod(Nat.1, p)
    }
}

/// A nonfixed reduced residue has normalized inverse different from `1`.
theorem prime_nonfixed_inv_not_one(p: Nat, x: Nat) {
    p.is_prime and coprime_residues(p).contains(x) and x != Nat.1 and x != p - Nat.1
        implies inv_mod_fn(p)(x) != Nat.1
} by {
    if p.is_prime and coprime_residues(p).contains(x) and x != Nat.1 and x != p - Nat.1 {
        Nat.1 < p
        p != Nat.0
        coprime_residues_contains_imp(p, x)
        x < p and x.coprime(p)
        x < p
        x.coprime(p)
        let ix: Nat = inv_mod_fn(p)(x)
        inv_mod_fn_mem(p, x)
        coprime_residues(p).contains(ix)
        coprime_residues_contains_imp(p, ix)
        ix < p and ix.coprime(p)
        ix < p
        if ix = Nat.1 {
            inv_mod_fn_mul_congr_one(p, x)
            (x * ix).congr_mod(Nat.1, p)
            x * ix = x * Nat.1
            x * Nat.1 = x
            x.congr_mod(Nat.1, p)
            congr_mod_below_eq(p, x, Nat.1)
            x = Nat.1
            false
        }
        inv_mod_fn(p)(x) != Nat.1
    }
}

/// A nonfixed reduced residue has normalized inverse different from `p - 1`.
theorem prime_nonfixed_inv_not_pred(p: Nat, x: Nat) {
    p.is_prime and coprime_residues(p).contains(x) and x != Nat.1 and x != p - Nat.1
        implies inv_mod_fn(p)(x) != p - Nat.1
} by {
    if p.is_prime and coprime_residues(p).contains(x) and x != Nat.1 and x != p - Nat.1 {
        Nat.1 < p
        p != Nat.0
        coprime_residues_contains_imp(p, x)
        x < p and x.coprime(p)
        x < p
        x.coprime(p)
        let ix: Nat = inv_mod_fn(p)(x)
        inv_mod_fn_mem(p, x)
        coprime_residues(p).contains(ix)
        if ix = p - Nat.1 {
            inv_mod_fn_mul_congr_one(p, x)
            (x * ix).congr_mod(Nat.1, p)
            x * ix = x * (p - Nat.1)
            x * (p - Nat.1) = (p - Nat.1) * x
            ((p - Nat.1) * x).congr_mod(Nat.1, p)
            prime_pred_self_inverse_congr(p)
            ((p - Nat.1) * (p - Nat.1)).congr_mod(Nat.1, p)
            mod_inv_unique_congr(p - Nat.1, p, x, p - Nat.1)
            x.congr_mod(p - Nat.1, p)
            sub_one_lt(p)
            p - Nat.1 < p
            congr_mod_below_eq(p, x, p - Nat.1)
            x = p - Nat.1
            false
        }
        inv_mod_fn(p)(x) != p - Nat.1
    }
}

/// A nonfixed reduced residue has normalized inverse different from `1` and
/// `p - 1`.
theorem prime_nonfixed_inv_not_one_or_pred(p: Nat, x: Nat) {
    p.is_prime and coprime_residues(p).contains(x) and x != Nat.1 and x != p - Nat.1
        implies inv_mod_fn(p)(x) != Nat.1 and inv_mod_fn(p)(x) != p - Nat.1
} by {
    if p.is_prime and coprime_residues(p).contains(x) and x != Nat.1 and x != p - Nat.1 {
        prime_nonfixed_inv_not_one(p, x)
        inv_mod_fn(p)(x) != Nat.1
        prime_nonfixed_inv_not_pred(p, x)
        inv_mod_fn(p)(x) != p - Nat.1
        inv_mod_fn(p)(x) != Nat.1 and inv_mod_fn(p)(x) != p - Nat.1
    }
}

/// After removing `1` and `p - 1`, the remaining reduced residues are pairable
/// by normalized inversion.
theorem wilson_nonfixed_residues_pairable(p: Nat) {
    p.is_prime and Nat.2 < p implies wilson_pairable_list(p, wilson_nonfixed_residues(p))
} by {
    if p.is_prime and Nat.2 < p {
        let all: List[Nat] = coprime_residues(p)
        let without_one: List[Nat] = all.remove_one(Nat.1)
        let without_fixed: List[Nat] = without_one.remove_one(p - Nat.1)
        Nat.1 < p
        p != Nat.0
        prime_one_ne_pred_of_gt_two(p)
        Nat.1 != p - Nat.1
        p - Nat.1 != Nat.1
        coprime_residues_unique(p)
        all.is_unique
        remove_one_unique(all, Nat.1)
        without_one.is_unique
        remove_one_unique(without_one, p - Nat.1)
        without_fixed.is_unique
        remove_one_unique_not_contains_self(all, Nat.1)
        not without_one.contains(Nat.1)
        remove_one_unique_not_contains_self(without_one, p - Nat.1)
        not without_fixed.contains(p - Nat.1)
        wilson_nonfixed_residues(p) = without_fixed
        forall(x: Nat) {
            if without_fixed.contains(x) {
                remove_one_unique_contains_imp_contains_nat(without_one, p - Nat.1, x)
                without_one.contains(x)
                remove_one_unique_contains_imp_contains_nat(all, Nat.1, x)
                all.contains(x)
                coprime_residues(p).contains(x)
            }
        }
        wilson_items_are_residues(p, without_fixed)
        forall(x: Nat) {
            if without_fixed.contains(x) {
                if x = p - Nat.1 {
                    without_fixed.contains(p - Nat.1)
                    false
                }
                x != p - Nat.1
                if x = Nat.1 {
                    remove_one_contains_other(without_one, p - Nat.1, Nat.1)
                    without_one.contains(Nat.1)
                    false
                }
                x != Nat.1
                x != Nat.1 and x != p - Nat.1
            }
        }
        wilson_items_nonfixed(p, without_fixed)
        forall(x: Nat) {
            if without_fixed.contains(x) {
                remove_one_unique_contains_imp_contains_nat(without_one, p - Nat.1, x)
                without_one.contains(x)
                remove_one_unique_contains_imp_contains_nat(all, Nat.1, x)
                all.contains(x)
                coprime_residues(p).contains(x)
                if x = Nat.1 {
                    remove_one_contains_other(without_one, p - Nat.1, Nat.1)
                    without_one.contains(Nat.1)
                    false
                }
                x != Nat.1
                if x = p - Nat.1 {
                    without_fixed.contains(p - Nat.1)
                    false
                }
                x != p - Nat.1
                prime_nonfixed_inv_not_one_or_pred(p, x)
                inv_mod_fn(p)(x) != Nat.1 and inv_mod_fn(p)(x) != p - Nat.1
                let ix: Nat = inv_mod_fn(p)(x)
                inv_mod_fn_mem(p, x)
                all.contains(ix)
                remove_one_contains_other(all, Nat.1, ix)
                without_one.contains(ix)
                remove_one_contains_other(without_one, p - Nat.1, ix)
                without_fixed.contains(ix)
                without_fixed.contains(inv_mod_fn(p)(x))
            }
        }
        wilson_items_inv_closed(p, without_fixed)
        wilson_pairable_list(p, without_fixed)
        wilson_pairable_list(p, wilson_nonfixed_residues(p))
    }
}

/// A nontrivial reduced residue modulo a prime is not fixed by normalized
/// inversion.
theorem prime_nontrivial_inv_not_fixed(p: Nat, x: Nat) {
    p.is_prime and coprime_residues(p).contains(x) and x != Nat.1 and x != p - Nat.1
        implies inv_mod_fn(p)(x) != x
} by {
    if p.is_prime and coprime_residues(p).contains(x) and x != Nat.1 and x != p - Nat.1 {
        if inv_mod_fn(p)(x) = x {
            prime_inv_fixed_eq_one_or_pred(p, x)
            x = Nat.1 or x = p - Nat.1
            false
        }
    }
}

/// If normalized inversion of `x` is `y`, then normalized inversion of `y` is
/// `x`, for reduced residues.
theorem inv_mod_fn_swap_on_residues(n: Nat, x: Nat, y: Nat) {
    n != Nat.0 and coprime_residues(n).contains(x) and inv_mod_fn(n)(x) = y
        implies inv_mod_fn(n)(y) = x
} by {
    if n != Nat.0 and coprime_residues(n).contains(x) and inv_mod_fn(n)(x) = y {
        inv_mod_fn_involutive_on_residues(n, x)
        inv_mod_fn(n)(inv_mod_fn(n)(x)) = x
        inv_mod_fn(n)(y) = x
    }
}

/// Removing the inverse of a pairable head leaves a unique tail.
theorem wilson_remove_pair_unique(p: Nat, head: Nat, tail: List[Nat]) {
    wilson_pairable_list(p, List.cons(head, tail))
        implies tail.remove_one(inv_mod_fn(p)(head)).is_unique
} by {
    if wilson_pairable_list(p, List.cons(head, tail)) {
        let items: List[Nat] = List.cons(head, tail)
        let y: Nat = inv_mod_fn(p)(head)
        wilson_pairable_unique(p, items)
        items.is_unique
        unique_implies_tail_unique(head, tail)
        tail.is_unique
        remove_one_unique(tail, y)
    }
}

/// Removing the inverse of a pairable head preserves the residue condition.
theorem wilson_remove_pair_residues(p: Nat, head: Nat, tail: List[Nat]) {
    wilson_pairable_list(p, List.cons(head, tail))
        implies wilson_items_are_residues(p, tail.remove_one(inv_mod_fn(p)(head)))
} by {
    if wilson_pairable_list(p, List.cons(head, tail)) {
        let items: List[Nat] = List.cons(head, tail)
        let y: Nat = inv_mod_fn(p)(head)
        let rest: List[Nat] = tail.remove_one(y)
        wilson_pairable_unique(p, items)
        items.is_unique
        unique_implies_tail_unique(head, tail)
        tail.is_unique
        forall(x: Nat) {
            if rest.contains(x) {
                remove_one_unique_contains_imp_contains_nat(tail, y, x)
                tail.contains(x)
                items.contains(x)
                wilson_pairable_contains_residue(p, items, x)
                coprime_residues(p).contains(x)
            }
        }
        wilson_items_are_residues(p, rest)
        wilson_items_are_residues(p, tail.remove_one(inv_mod_fn(p)(head)))
    }
}

/// Removing the inverse of a pairable head preserves exclusion of fixed residues.
theorem wilson_remove_pair_nonfixed(p: Nat, head: Nat, tail: List[Nat]) {
    wilson_pairable_list(p, List.cons(head, tail))
        implies wilson_items_nonfixed(p, tail.remove_one(inv_mod_fn(p)(head)))
} by {
    if wilson_pairable_list(p, List.cons(head, tail)) {
        let items: List[Nat] = List.cons(head, tail)
        let y: Nat = inv_mod_fn(p)(head)
        let rest: List[Nat] = tail.remove_one(y)
        wilson_pairable_unique(p, items)
        items.is_unique
        unique_implies_tail_unique(head, tail)
        tail.is_unique
        forall(x: Nat) {
            if rest.contains(x) {
                remove_one_unique_contains_imp_contains_nat(tail, y, x)
                tail.contains(x)
                items.contains(x)
                wilson_pairable_nonfixed(p, items, x)
                x != Nat.1 and x != p - Nat.1
            }
        }
        wilson_items_nonfixed(p, rest)
        wilson_items_nonfixed(p, tail.remove_one(inv_mod_fn(p)(head)))
    }
}

/// Removing the inverse of a pairable head preserves inverse-closure.
theorem wilson_remove_pair_inv_closed(p: Nat, head: Nat, tail: List[Nat]) {
    p.is_prime and wilson_pairable_list(p, List.cons(head, tail))
        implies wilson_items_inv_closed(p, tail.remove_one(inv_mod_fn(p)(head)))
} by {
    if p.is_prime and wilson_pairable_list(p, List.cons(head, tail)) {
        let items: List[Nat] = List.cons(head, tail)
        let y: Nat = inv_mod_fn(p)(head)
        let rest: List[Nat] = tail.remove_one(y)
        Nat.1 < p
        p != Nat.0
        wilson_pairable_unique(p, items)
        items.is_unique
        unique_implies_tail_unique(head, tail)
        tail.is_unique
        remove_one_unique_not_contains_self(tail, y)
        not rest.contains(y)
        unique_cons_not_tail_nat(head, tail)
        not tail.contains(head)
        items.contains(head)
        wilson_pairable_contains_residue(p, items, head)
        coprime_residues(p).contains(head)
        forall(x: Nat) {
            if rest.contains(x) {
                remove_one_unique_contains_imp_ne_item_nat(tail, y, x)
                x != y
                remove_one_unique_contains_imp_contains_nat(tail, y, x)
                tail.contains(x)
                items.contains(x)
                wilson_pairable_contains_residue(p, items, x)
                coprime_residues(p).contains(x)
                wilson_pairable_inv_mem(p, items, x)
                items.contains(inv_mod_fn(p)(x))
                let ix: Nat = inv_mod_fn(p)(x)
                if ix = head {
                    inv_mod_fn_swap_on_residues(p, x, head)
                    inv_mod_fn(p)(head) = x
                    y = x
                    rest.contains(y)
                    false
                }
                ix != head
                if ix = y {
                    inv_mod_fn_swap_on_residues(p, x, y)
                    inv_mod_fn(p)(y) = x
                    inv_mod_fn_swap_on_residues(p, head, y)
                    inv_mod_fn(p)(y) = head
                    x = head
                    tail.contains(head)
                    false
                }
                ix != y
                tail.contains(ix)
                remove_one_contains_other(tail, y, ix)
                rest.contains(ix)
                rest.contains(inv_mod_fn(p)(x))
            }
        }
        forall(x: Nat) { rest.contains(x) implies rest.contains(inv_mod_fn(p)(x)) }
        wilson_items_inv_closed(p, rest)
        wilson_items_inv_closed(p, tail.remove_one(inv_mod_fn(p)(head)))
    }
}

/// Removing a nontrivial head together with its inverse preserves pairability
/// for the remaining tail.
theorem wilson_pairable_remove_inverse_pair(p: Nat, head: Nat, tail: List[Nat]) {
    p.is_prime and wilson_pairable_list(p, List.cons(head, tail))
        implies wilson_pairable_list(p, tail.remove_one(inv_mod_fn(p)(head)))
} by {
    if p.is_prime and wilson_pairable_list(p, List.cons(head, tail)) {
        let rest: List[Nat] = tail.remove_one(inv_mod_fn(p)(head))
        wilson_remove_pair_unique(p, head, tail)
        rest.is_unique
        wilson_remove_pair_residues(p, head, tail)
        wilson_items_are_residues(p, rest)
        wilson_remove_pair_nonfixed(p, head, tail)
        wilson_items_nonfixed(p, rest)
        wilson_remove_pair_inv_closed(p, head, tail)
        wilson_items_inv_closed(p, rest)
        wilson_pairable_list(p, rest)
        wilson_pairable_list(p, tail.remove_one(inv_mod_fn(p)(head)))
    }
}

/// Bounded induction predicate for pairable-list products.
define wilson_pairable_product_bound_pred(p: Nat) -> (Nat -> Bool) {
    function(m: Nat) {
        forall(items: List[Nat]) {
            items.length <= m and p.is_prime and wilson_pairable_list(p, items)
                implies product[Nat](items).congr_mod(Nat.1, p)
        }
    }
}

/// Apply the bounded pairable-product induction predicate at one list.
theorem wilson_pairable_product_bound_apply(p: Nat, m: Nat, items: List[Nat]) {
    wilson_pairable_product_bound_pred(p)(m) and items.length <= m and
    p.is_prime and wilson_pairable_list(p, items)
        implies product[Nat](items).congr_mod(Nat.1, p)
} by {
    if wilson_pairable_product_bound_pred(p)(m) and items.length <= m and
        p.is_prime and wilson_pairable_list(p, items) {
        wilson_pairable_product_bound_pred(p)(m) = forall(xs: List[Nat]) { xs.length <= m and p.is_prime and wilson_pairable_list(p, xs) implies product[Nat](xs).congr_mod(Nat.1, p) }
        forall(xs: List[Nat]) { xs.length <= m and p.is_prime and wilson_pairable_list(p, xs) implies product[Nat](xs).congr_mod(Nat.1, p) }
        product[Nat](items).congr_mod(Nat.1, p)
    }
}

/// Strong-induction step for pairable-list products.
theorem wilson_pairable_product_bound_step(p: Nat, m: Nat) {
    true_below(wilson_pairable_product_bound_pred(p), m)
        implies wilson_pairable_product_bound_pred(p)(m)
} by {
    if true_below(wilson_pairable_product_bound_pred(p), m) {
        forall(items: List[Nat]) {
            if items.length <= m and p.is_prime and wilson_pairable_list(p, items) {
                match items {
                    List.nil {
                        product[Nat](List.nil[Nat]) = Nat.1
                        congr_mod_refl(Nat.1, p)
                        product[Nat](items).congr_mod(Nat.1, p)
                    }
                    List.cons(head, tail) {
                        let y: Nat = inv_mod_fn(p)(head)
                        let rest: List[Nat] = tail.remove_one(y)
                        Nat.1 < p
                        p != Nat.0
                        items = List.cons(head, tail)
                        items.contains(head)
                        wilson_pairable_contains_residue(p, items, head)
                        coprime_residues(p).contains(head)
                        coprime_residues_contains_imp(p, head)
                        head < p and head.coprime(p)
                        head.coprime(p)
                        wilson_pairable_nonfixed(p, items, head)
                        head != Nat.1 and head != p - Nat.1
                        prime_nontrivial_inv_not_fixed(p, head)
                        y != head
                        head != y
                        wilson_pairable_inv_mem(p, items, head)
                        items.contains(y)
                        tail.contains(y)
                        product_remove_one(tail, y)
                        y * product[Nat](rest) = product[Nat](tail)
                        product[Nat](items) = head * product[Nat](tail)
                        product[Nat](items) = head * (y * product[Nat](rest))
                        head * (y * product[Nat](rest)) = (head * y) * product[Nat](rest)
                        product[Nat](items) = (head * y) * product[Nat](rest)
                        wilson_pairable_remove_inverse_pair(p, head, tail)
                        wilson_pairable_list(p, rest)
                        remove_one_tail_length_lt_cons(head, tail, y)
                        rest.length < List.cons(head, tail).length
                        items.length = List.cons(head, tail).length
                        rest.length < items.length
                        rest.length < m
                        let pred: Nat -> Bool = wilson_pairable_product_bound_pred(p)
                        let h: Bool = rest.length < m implies pred(rest.length)
                        h
                        pred(rest.length)
                        wilson_pairable_product_bound_pred(p)(rest.length)
                        rest.length <= rest.length
                        wilson_pairable_product_bound_apply(p, rest.length, rest)
                        product[Nat](rest).congr_mod(Nat.1, p)
                        inv_mod_fn_mul_congr_one(p, head)
                        (head * y).congr_mod(Nat.1, p)
                        congr_mod_mul(head * y, product[Nat](rest), Nat.1, Nat.1, p)
                        ((head * y) * product[Nat](rest)).congr_mod(Nat.1 * Nat.1, p)
                        Nat.1 * Nat.1 = Nat.1
                        ((head * y) * product[Nat](rest)).congr_mod(Nat.1, p)
                        product[Nat](items).congr_mod(Nat.1, p)
                    }
                }
            }
        }
        wilson_pairable_product_bound_pred(p)(m)
    }
}

/// Products of pairable residue lists are congruent to one.
theorem wilson_pairable_product_congr_one(p: Nat, items: List[Nat]) {
    p.is_prime and wilson_pairable_list(p, items)
        implies product[Nat](items).congr_mod(Nat.1, p)
} by {
    let pred: Nat -> Bool = wilson_pairable_product_bound_pred(p)
    forall(m: Nat) {
        if true_below(pred, m) {
            wilson_pairable_product_bound_step(p, m)
            pred(m)
        }
    }
    strong_induction(pred)
    forall(m: Nat) { pred(m) }
    pred(items.length)
    wilson_pairable_product_bound_pred(p)(items.length)
    if p.is_prime and wilson_pairable_list(p, items) {
        items.length <= items.length
        product[Nat](items).congr_mod(Nat.1, p)
    }
}

/// For a prime modulus, the product of all reduced residues is congruent either
/// to `1` or to `p - 1`. Wilson's prime direction is the remaining exclusion of
/// the `1` case, obtained by pairing non-fixed inverse orbits.
theorem prime_product_residues_mod_eq_one_or_pred(p: Nat) {
    p.is_prime implies
        product[Nat](coprime_residues(p)).mod(p) = Nat.1 or
        product[Nat](coprime_residues(p)).mod(p) = p - Nat.1
} by {
    if p.is_prime {
        let residue_product: Nat = product[Nat](coprime_residues(p))
        let residue_mod: Nat = residue_product.mod(p)
        Nat.1 < p
        p != Nat.0
        product_coprime_residues_square_congr_one(p)
        (residue_product * residue_product).congr_mod(Nat.1, p)
        product_coprime_residues_coprime(p)
        residue_product.coprime(p)
        coprime_mod_imp(residue_product, p)
        residue_mod.coprime(p)
        mod_lt(residue_product, p)
        residue_mod < p
        mod_congr_mod_self(residue_product, p)
        residue_mod.congr_mod(residue_product, p)
        congr_mod_mul(residue_mod, residue_mod, residue_product, residue_product, p)
        (residue_mod * residue_mod).congr_mod(residue_product * residue_product, p)
        congr_mod_trans(residue_mod * residue_mod, residue_product * residue_product, Nat.1, p)
        (residue_mod * residue_mod).congr_mod(Nat.1, p)
        prime_self_inverse_residue_eq_one_or_pred(p, residue_mod)
        residue_mod = Nat.1 or residue_mod = p - Nat.1
        if residue_mod = Nat.1 {
            product[Nat](coprime_residues(p)).mod(p) = Nat.1
            product[Nat](coprime_residues(p)).mod(p) = Nat.1 or
                product[Nat](coprime_residues(p)).mod(p) = p - Nat.1
        } else {
            residue_mod = p - Nat.1
            product[Nat](coprime_residues(p)).mod(p) = p - Nat.1
            product[Nat](coprime_residues(p)).mod(p) = Nat.1 or
                product[Nat](coprime_residues(p)).mod(p) = p - Nat.1
        }
    }
}

/// Factorial at a positive natural splits off that natural.
theorem factorial_split_pred(n: Nat) {
    n != Nat.0 implies n.factorial = n * (n - Nat.1).factorial
} by {
    if n != Nat.0 {
        let pred: Nat satisfy { pred.suc = n }
        n = pred.suc
        n - Nat.1 = pred
        factorial_step(pred)
        pred.suc.factorial = pred.suc * pred.factorial
        n.factorial = n * (n - Nat.1).factorial
    }
}

/// Inductive step for the product of reduced residues below a prime.
theorem prime_coprime_residues_product_step(p: Nat, k: Nat) {
    p.is_prime and prime_coprime_residues_product_pred(p)(k)
        implies prime_coprime_residues_product_pred(p)(k.suc)
} by {
    if p.is_prime and prime_coprime_residues_product_pred(p)(k) {
        if k.suc <= p {
            k <= p
            product[Nat](coprime_residues_below(p, k)) = (k - Nat.1).factorial
            if k = Nat.0 {
                zero_not_coprime_prime(p)
                not k.coprime(p)
                coprime_residues_below_suc_no(p, k)
                coprime_residues_below(p, k.suc) = coprime_residues_below(p, k)
                coprime_residues_below(p, k) = List.nil[Nat]
                product[Nat](coprime_residues_below(p, k.suc)) =
                    product[Nat](List.nil[Nat])
                product[Nat](List.nil[Nat]) = Nat.1
                k.suc - Nat.1 = Nat.0
                (k.suc - Nat.1).factorial = Nat.0.factorial
                Nat.0.factorial = Nat.1
                product[Nat](coprime_residues_below(p, k.suc)) = (k.suc - Nat.1).factorial
            } else {
                Nat.1 <= k
                k < k.suc
                k < p
                coprime_below_prime(p, k)
                k.coprime(p)
                coprime_residues_below_suc_yes(p, k)
                coprime_residues_below(p, k.suc) =
                    List.cons(k, coprime_residues_below(p, k))
                product[Nat](coprime_residues_below(p, k.suc)) =
                    k * product[Nat](coprime_residues_below(p, k))
                product[Nat](coprime_residues_below(p, k.suc)) =
                    k * (k - Nat.1).factorial
                factorial_split_pred(k)
                k.factorial = k * (k - Nat.1).factorial
                product[Nat](coprime_residues_below(p, k.suc)) = k.factorial
                suc_sub_one(k)
                k.suc - Nat.1 = k
                product[Nat](coprime_residues_below(p, k.suc)) =
                    (k.suc - Nat.1).factorial
            }
        }
    }
}

/// Inner induction for the product of reduced residues below a prime.
theorem prime_coprime_residues_product_run(p: Nat, k: Nat) {
    p.is_prime implies prime_coprime_residues_product_pred(p)(k)
} by {
    if p.is_prime {
        let f: Nat -> Bool = function(x: Nat) {
            x <= p implies
                product[Nat](coprime_residues_below(p, x)) = (x - Nat.1).factorial
        }
        forall(x: Nat) {
            prime_coprime_residues_product_pred(p)(x) = f(x)
            f(x) = prime_coprime_residues_product_pred(p)(x)
        }
        prime_coprime_residues_product_zero(p)
        f(Nat.0)
        forall(x: Nat) {
            if f(x) {
                prime_coprime_residues_product_pred(p)(x)
                prime_coprime_residues_product_step(p, x)
                prime_coprime_residues_product_pred(p)(x.suc)
                f(x.suc)
            }
        }
        f(k)
        prime_coprime_residues_product_pred(p)(k)
    }
}

/// For a prime `p`, the product of the coprime residues below each `k <= p`
/// is `(k - 1)!`.
theorem prime_coprime_residues_below_product_factorial(p: Nat, k: Nat) {
    p.is_prime and k <= p implies
        product[Nat](coprime_residues_below(p, k)) = (k - Nat.1).factorial
} by {
    if p.is_prime and k <= p {
        prime_coprime_residues_product_run(p, k)
        prime_coprime_residues_product_pred(p)(k)
    }
}

/// For a prime `p`, the product of all reduced residues modulo `p` is `(p - 1)!`.
theorem prime_coprime_residues_product_factorial(p: Nat) {
    p.is_prime implies product[Nat](coprime_residues(p)) = (p - Nat.1).factorial
} by {
    if p.is_prime {
        p <= p
        prime_coprime_residues_below_product_factorial(p, p)
        product[Nat](coprime_residues_below(p, p)) = (p - Nat.1).factorial
        coprime_residues(p) = coprime_residues_below(p, p)
        product[Nat](coprime_residues(p)) = (p - Nat.1).factorial
    }
}

/// For primes greater than two, the product of all reduced residues splits into
/// the two fixed residues and the nonfixed part.
theorem prime_gt_two_product_residues_decomp(p: Nat) {
    p.is_prime and Nat.2 < p implies
        product[Nat](coprime_residues(p)) =
        (p - Nat.1) * product[Nat](wilson_nonfixed_residues(p))
} by {
    if p.is_prime and Nat.2 < p {
        let all: List[Nat] = coprime_residues(p)
        let without_one: List[Nat] = all.remove_one(Nat.1)
        let without_fixed: List[Nat] = without_one.remove_one(p - Nat.1)
        Nat.1 < p
        p != Nat.0
        prime_one_ne_pred_of_gt_two(p)
        Nat.1 != p - Nat.1
        p - Nat.1 != Nat.1
        prime_one_residue(p)
        all.contains(Nat.1)
        product_remove_one[Nat](all, Nat.1)
        Nat.1 * product[Nat](without_one) = product[Nat](all)
        Nat.1 * product[Nat](without_one) = product[Nat](without_one)
        product[Nat](all) = product[Nat](without_one)
        prime_pred_residue(p)
        all.contains(p - Nat.1)
        remove_one_contains_other(all, Nat.1, p - Nat.1)
        without_one.contains(p - Nat.1)
        product_remove_one[Nat](without_one, p - Nat.1)
        (p - Nat.1) * product[Nat](without_fixed) = product[Nat](without_one)
        wilson_nonfixed_residues(p) = without_fixed
        product[Nat](all) = (p - Nat.1) * product[Nat](wilson_nonfixed_residues(p))
        product[Nat](coprime_residues(p)) =
            (p - Nat.1) * product[Nat](wilson_nonfixed_residues(p))
    }
}

/// For primes greater than two, the product of all reduced residues is
/// congruent to `p - 1` modulo `p`.
theorem prime_gt_two_product_residues_congr_pred(p: Nat) {
    p.is_prime and Nat.2 < p implies
        product[Nat](coprime_residues(p)).congr_mod(p - Nat.1, p)
} by {
    if p.is_prime and Nat.2 < p {
        let nonfixed: List[Nat] = wilson_nonfixed_residues(p)
        prime_gt_two_product_residues_decomp(p)
        product[Nat](coprime_residues(p)) = (p - Nat.1) * product[Nat](nonfixed)
        wilson_nonfixed_residues_pairable(p)
        wilson_pairable_list(p, nonfixed)
        wilson_pairable_product_congr_one(p, nonfixed)
        product[Nat](nonfixed).congr_mod(Nat.1, p)
        congr_mod_refl(p - Nat.1, p)
        (p - Nat.1).congr_mod(p - Nat.1, p)
        congr_mod_mul(p - Nat.1, product[Nat](nonfixed), p - Nat.1, Nat.1, p)
        ((p - Nat.1) * product[Nat](nonfixed)).congr_mod((p - Nat.1) * Nat.1, p)
        (p - Nat.1) * Nat.1 = p - Nat.1
        ((p - Nat.1) * product[Nat](nonfixed)).congr_mod(p - Nat.1, p)
        product[Nat](coprime_residues(p)).congr_mod(p - Nat.1, p)
    }
}

/// For any prime, the product of all reduced residues is congruent to `p - 1`
/// modulo `p`.
theorem prime_product_residues_congr_pred(p: Nat) {
    p.is_prime implies product[Nat](coprime_residues(p)).congr_mod(p - Nat.1, p)
} by {
    if p.is_prime {
        Nat.1 < p
        one_lt_eq_two_or_two_lt(p)
        p = Nat.2 or Nat.2 < p
        if p = Nat.2 {
            prime_coprime_residues_product_factorial(p)
            product[Nat](coprime_residues(p)) = (p - Nat.1).factorial
            p - Nat.1 = Nat.1
            (p - Nat.1).factorial = Nat.1
            product[Nat](coprime_residues(p)) = p - Nat.1
            congr_mod_refl(p - Nat.1, p)
            product[Nat](coprime_residues(p)).congr_mod(p - Nat.1, p)
        } else {
            Nat.2 < p
            prime_gt_two_product_residues_congr_pred(p)
            product[Nat](coprime_residues(p)).congr_mod(p - Nat.1, p)
        }
    }
}

/// Wilson's factorial congruence holds for prime moduli.
theorem prime_imp_wilson_factorial_congr(p: Nat) {
    p.is_prime implies (p - Nat.1).factorial.congr_mod(p - Nat.1, p)
} by {
    if p.is_prime {
        prime_coprime_residues_product_factorial(p)
        product[Nat](coprime_residues(p)) = (p - Nat.1).factorial
        prime_product_residues_congr_pred(p)
        product[Nat](coprime_residues(p)).congr_mod(p - Nat.1, p)
        (p - Nat.1).factorial.congr_mod(p - Nat.1, p)
    }
}

/// A composite modulus cannot satisfy Wilson's factorial congruence.
theorem composite_not_wilson_factorial_congr(p: Nat) {
    p.is_composite implies not (p - Nat.1).factorial.congr_mod(p - Nat.1, p)
} by {
    if p.is_composite {
        let (b: Nat, c: Nat) satisfy {
            Nat.1 < b and Nat.1 < c and p = b * c
        }
        b != Nat.0
        b.divides(p)
        divisor_lt(b, c, p)
        b < p
        lt_imp_le_sub_one(b, p)
        b <= p - Nat.1
        divides_factorial(b, p - Nat.1)
        b.divides((p - Nat.1).factorial)
        if (p - Nat.1).factorial.congr_mod(p - Nat.1, p) {
            divides_mod((p - Nat.1).factorial, p, b)
            b.divides((p - Nat.1).factorial.mod(p))
            (p - Nat.1).factorial.mod(p) = (p - Nat.1).mod(p)
            Nat.0 < p
            sub_one_lt(p)
            p - Nat.1 < p
            small_mod(p - Nat.1, p)
            (p - Nat.1).mod(p) = p - Nat.1
            b.divides(p - Nat.1)
            divides_sub(p, p - Nat.1, b)
            b.divides(p - (p - Nat.1))
            p != Nat.0
            sub_pred_eq_one(p)
            p - (p - Nat.1) = Nat.1
            b.divides(Nat.1)
            nat_divides_one_imp_one(b)
            b = Nat.1
            false
        }
    }
}

/// Zero does not satisfy Wilson's factorial congruence.
theorem zero_not_wilson_factorial_congr {
    not (Nat.0 - Nat.1).factorial.congr_mod(Nat.0 - Nat.1, Nat.0)
} by {
    Nat.0 - Nat.1 = Nat.0
    Nat.0.factorial = Nat.1
    mod_by_zero(Nat.1)
    Nat.1.mod(Nat.0) = Nat.1
    mod_by_zero(Nat.0)
    Nat.0.mod(Nat.0) = Nat.0
    if (Nat.0 - Nat.1).factorial.congr_mod(Nat.0 - Nat.1, Nat.0) {
        Nat.1.mod(Nat.0) = Nat.0.mod(Nat.0)
        Nat.1 = Nat.0
        false
    }
}

/// Any natural other than zero and one is greater than one.
theorem nonzero_ne_one_imp_one_lt(p: Nat) {
    p != Nat.0 and p != Nat.1 implies Nat.1 < p
} by {
    if p != Nat.0 and p != Nat.1 {
        if p < Nat.1 {
            p = Nat.0
            false
        }
        if p = Nat.1 {
            false
        }
        Nat.1 < p
    }
}

/// Wilson's congruence implies primality. This is the easier half of Wilson's
/// theorem; the prime-to-congruence direction still needs the inverse-pairing
/// product argument.
theorem wilson_factorial_congr_imp_prime(p: Nat) {
    p != Nat.1 and (p - Nat.1).factorial.congr_mod(p - Nat.1, p)
        implies p.is_prime
} by {
    if p != Nat.1 and (p - Nat.1).factorial.congr_mod(p - Nat.1, p) {
        if p = Nat.0 {
            zero_not_wilson_factorial_congr
            false
        }
        p != Nat.0
        nonzero_ne_one_imp_one_lt(p)
        Nat.1 < p
        if not p.is_prime {
            p.is_composite
            composite_not_wilson_factorial_congr(p)
            false
        }
    }
}
