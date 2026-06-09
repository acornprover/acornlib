from nat import Nat, from_nat, add_sub, lt_and_lte, lt_or_lte, lte_antisymm,
    lte_imp_not_lt, sub_pos, sub_one_lt, false_below, false_below_apply,
    is_min, is_min_apply, is_min_false_below, has_min, lt_suc, mul_two_left,
    lt_imp_lte_suc, true_below, true_below_apply, true_below_zero,
    true_below_suc_intro, zero_or_suc, lte_cancel_suc, alt_induction
from int import Int, abs, add_sub_lt_right_of_lt, from_nat_pos, mul_pos_pos,
    sub_nonnegative_of_lte, zero_not_pos, nonzero_pos_or_neg,
    lte_and_lt, neg_lt_nonneg, lt_not_ref,
    lte_nonnegative_ints_implies_nats, member_abs_pos
from rat import Rat, nat_lt_imp_rat_lt, zero_lt_imp_pos, pos_inverse, pos_ne_zero,
    recip_eq_one_div, lte_add_left, lte_add_right, lte_trans, pos_lte, cross_mul_lte,
    nat_lte_imp_rat_lte, mul_one_left, smaller_int_inverse,
    positive_int_eq_from_nat_suc, gt_minus_pos, rat_total, lt_trans, lt_lte_trans,
    lte_lt_trans, lt_cancel_add_right, not_lt_self,
    denom_positive, lte_mul_pos, mul_denom, from_int_lte_cancel, mul_int_eq_int_mul,
    from_int_lt_cancel, mul_inv_cancels_left, mul_comm, lt_mul_pos, reduce,
    from_int_num, from_int_denom, reduce_idempotent, add_reduced, reduce_neg_num,
    denom_nonzero, reduce_num_lte_raw_of_positive, times_two, from_nat_mul,
    pos_imp_zero_lt, zero_recip, reduce_zero_num
from order import lt_of_lte_of_lt
from list import List, map, sum, sum_add, map_add, singleton_unique,
    is_lower_bound, is_upper_bound, lower_bound_nil, upper_bound_nil,
    lower_bound_cons_iff, upper_bound_cons_iff, lower_bound_add_iff,
    upper_bound_add_iff, lower_bound_monotone, lower_bound_singleton_iff,
    list_lower_bound, list_upper_bound,
    list_lower_bound_imp_lower_bound, list_upper_bound_imp_upper_bound,
    lower_bound_contains, upper_bound_contains,
    filter_contained_by_and, cons_unique_of_tail_unique_not_contains, unique_list_sum
from real import Real, add_from_rat, from_nat_is_from_rat, real_from_rat_inverse

numerals Nat

/// The rational unit fraction with denominator `n`.
/// By the ambient rational convention, the zero denominator gives zero.
define unit_fraction(n: Nat) -> Rat {
    Rat.from_nat(n).inverse
}

/// The real unit fraction with denominator `n`.
/// By the ambient real convention, the zero denominator gives zero.
define real_unit_fraction(n: Nat) -> Real {
    Real.1 / from_nat[Real](n)
}

/// The finite sum of unit fractions with the listed denominators.
define unit_fraction_sum(denominators: List[Nat]) -> Rat {
    sum(map(denominators, unit_fraction))
}

/// The finite real sum of unit fractions with the listed denominators.
define real_unit_fraction_sum(denominators: List[Nat]) -> Real {
    sum(map(denominators, real_unit_fraction))
}

/// True if every listed denominator is positive.
define positive_denominator_list(denominators: List[Nat]) -> Bool {
    match denominators {
        List.nil {
            true
        }
        List.cons(head, tail) {
            Nat.0 < head and positive_denominator_list(tail)
        }
    }
}

/// True if every listed denominator is at least `bound`.
define denominator_list_lower_bound(bound: Nat, denominators: List[Nat]) -> Bool {
    is_lower_bound(denominators, bound)
}

/// True if every listed denominator is at most `bound`.
define denominator_list_upper_bound(bound: Nat, denominators: List[Nat]) -> Bool {
    is_upper_bound(denominators, bound)
}

/// True if the denominators form a distinct positive list.
define egyptian_denominator_list(denominators: List[Nat]) -> Bool {
    denominators.is_unique and positive_denominator_list(denominators)
}

/// True if a rational is a finite sum of distinct unit fractions.
define is_egyptian_fraction(q: Rat) -> Bool {
    exists(denominators: List[Nat]) {
        egyptian_denominator_list(denominators) and q = unit_fraction_sum(denominators)
    }
}

/// True if a rational is represented by distinct unit fractions whose
/// denominators are all at least `bound`.
define is_egyptian_fraction_with_lower_bound(q: Rat, bound: Nat) -> Bool {
    exists(denominators: List[Nat]) {
        egyptian_denominator_list(denominators) and
        denominator_list_lower_bound(bound, denominators) and
        q = unit_fraction_sum(denominators)
    }
}

/// True if a rational has an Egyptian representation using only denominators
/// at least `bound`.
define is_lower_bounded_egyptian_fraction(q: Rat, bound: Nat) -> Bool {
    exists(denominators: List[Nat]) {
        egyptian_denominator_list(denominators) and
        denominator_list_lower_bound(bound, denominators) and
        q = unit_fraction_sum(denominators)
    }
}

/// True if a denominator gives a unit-fraction subtraction step for `q`.
define unit_fraction_greedy_step(q: Rat, n: Nat) -> Bool {
    Nat.0 < n and unit_fraction(n) < q and q - unit_fraction(n) < q
}

/// True if subtracting the unit fraction leaves a positive smaller remainder.
define unit_fraction_bounded_step(q: Rat, n: Nat) -> Bool {
    Nat.0 < n and unit_fraction(n) <= q and
        (q - unit_fraction(n)).is_positive and q - unit_fraction(n) < q
}

/// True if `n` is a positive denominator whose unit fraction is at most `q`.
define unit_fraction_bounded_denominator(q: Rat, n: Nat) -> Bool {
    Nat.0 < n and unit_fraction(n) <= q
}

/// The denominator predicate associated to a fixed rational.
define unit_fraction_bounded_denominator_for(q: Rat) -> (Nat -> Bool) {
    function(n: Nat) {
        unit_fraction_bounded_denominator(q, n)
    }
}

/// True if `n` is the least denominator whose unit fraction is at most `q`.
define unit_fraction_ceiling_step(q: Rat, n: Nat) -> Bool {
    is_min(unit_fraction_bounded_denominator_for(q), n)
}

/// The unreduced numerator of the remainder after subtracting `1/n` from `q`.
define unit_fraction_remainder_raw_num(q: Rat, n: Nat) -> Int {
    q.num * Int.from_nat(n) - q.denom
}

/// The unreduced denominator of the remainder after subtracting `1/n` from `q`.
define unit_fraction_remainder_raw_denom(q: Rat, n: Nat) -> Int {
    q.denom * Int.from_nat(n)
}

/// The natural measure of a rational used by the greedy-step descent.
define unit_fraction_num_measure(q: Rat) -> Nat {
    abs(q.num)
}

/// The natural measure of the unreduced remainder numerator.
define unit_fraction_remainder_raw_num_measure(q: Rat, n: Nat) -> Nat {
    abs(unit_fraction_remainder_raw_num(q, n))
}

/// True when the raw numerator of the remainder is a positive strict descent.
define unit_fraction_raw_num_descends(q: Rat, n: Nat) -> Bool {
    unit_fraction_remainder_raw_num(q, n).is_positive and
        unit_fraction_remainder_raw_num(q, n) < q.num
}

/// A positive denominator gives a positive unit fraction.
theorem unit_fraction_positive(n: Nat) {
    Nat.0 < n implies unit_fraction(n).is_positive
} by {
    if Nat.0 < n {
        nat_lt_imp_rat_lt(Nat.0, n)
        Rat.from_nat(Nat.0) < Rat.from_nat(n)
        Rat.from_nat(Nat.0) = Rat.0
        Rat.0 < Rat.from_nat(n)
        zero_lt_imp_pos(Rat.from_nat(n))
        Rat.from_nat(n).is_positive
        pos_inverse(Rat.from_nat(n))
        Rat.from_nat(n).inverse.is_positive
        unit_fraction(n).is_positive
    }
}

/// A positive denominator gives a nonzero unit fraction.
theorem unit_fraction_ne_zero(n: Nat) {
    Nat.0 < n implies unit_fraction(n) != Rat.0
} by {
    if Nat.0 < n {
        unit_fraction_positive(n)
        unit_fraction(n).is_positive
        pos_ne_zero(unit_fraction(n))
    }
}

/// For a positive denominator, the unit fraction is `1 / n`.
theorem unit_fraction_eq_one_div(n: Nat) {
    Nat.0 < n implies unit_fraction(n) = Rat.1 / Rat.from_nat(n)
} by {
    if Nat.0 < n {
        nat_lt_imp_rat_lt(Nat.0, n)
        Rat.from_nat(Nat.0) < Rat.from_nat(n)
        Rat.from_nat(Nat.0) = Rat.0
        Rat.0 < Rat.from_nat(n)
        zero_lt_imp_pos(Rat.from_nat(n))
        Rat.from_nat(n).is_positive
        pos_ne_zero(Rat.from_nat(n))
        Rat.from_nat(n) != Rat.0
        recip_eq_one_div(Rat.from_nat(n))
        Rat.from_nat(n).inverse = Rat.1 / Rat.from_nat(n)
        unit_fraction(n) = Rat.1 / Rat.from_nat(n)
    }
}

/// A positive unit fraction is the reduced fraction `1/n`.
theorem unit_fraction_eq_reduce(n: Nat) {
    Nat.0 < n implies unit_fraction(n) = reduce(Int.1, Int.from_nat(n))
} by {
    if Nat.0 < n {
        Rat.from_nat(n) = Rat.from_int(Int.from_nat(n))
        from_int_num(Int.from_nat(n))
        Rat.from_int(Int.from_nat(n)).num = Int.from_nat(n)
        from_int_denom(Int.from_nat(n))
        Rat.from_int(Int.from_nat(n)).denom = Int.1
        Rat.from_nat(n).num = Int.from_nat(n)
        Rat.from_nat(n).denom = Int.1
        unit_fraction(n) = Rat.from_nat(n).inverse
        Rat.from_nat(n).inverse = reduce(Rat.from_nat(n).denom, Rat.from_nat(n).num)
        Rat.from_nat(n).inverse = reduce(Int.1, Int.from_nat(n))
        unit_fraction(n) = reduce(Int.1, Int.from_nat(n))
    }
}

/// The first unit fraction is one.
theorem unit_fraction_one {
    unit_fraction(Nat.1) = Rat.1
} by {
    unit_fraction_eq_one_div(Nat.1)
    Rat.from_nat(Nat.1) = Rat.1
    Rat.1 / Rat.1 = Rat.1
}

/// The zero denominator gives the zero rational.
theorem unit_fraction_zero {
    unit_fraction(Nat.0) = Rat.0
} by {
    Rat.from_nat(Nat.0) = Rat.0
    zero_recip(Rat.0)
    Rat.0.inverse = Rat.0
    unit_fraction(Nat.0) = Rat.0.inverse
    unit_fraction(Nat.0) = Rat.0
}

/// Multiplying a positive unit fraction by its denominator gives one.
theorem unit_fraction_mul_denominator(n: Nat) {
    Nat.0 < n implies unit_fraction(n) * Rat.from_nat(n) = Rat.1
} by {
    if Nat.0 < n {
        nat_lt_imp_rat_lt(Nat.0, n)
        Rat.from_nat(Nat.0) < Rat.from_nat(n)
        Rat.from_nat(Nat.0) = Rat.0
        Rat.0 < Rat.from_nat(n)
        zero_lt_imp_pos(Rat.from_nat(n))
        Rat.from_nat(n).is_positive
        pos_ne_zero(Rat.from_nat(n))
        Rat.from_nat(n) != Rat.0
        mul_inv_cancels_left(Rat.from_nat(n))
        Rat.from_nat(n).inverse * Rat.from_nat(n) = Rat.1
        unit_fraction(n) = Rat.from_nat(n).inverse
        unit_fraction(n) * Rat.from_nat(n) = Rat.1
    }
}

/// A positive rational unit fraction embeds as the matching real unit fraction.
theorem unit_fraction_to_real(n: Nat) {
    Nat.0 < n implies Real.from_rat(unit_fraction(n)) = real_unit_fraction(n)
} by {
    if Nat.0 < n {
        nat_lt_imp_rat_lt(Nat.0, n)
        Rat.from_nat(Nat.0) < Rat.from_nat(n)
        Rat.from_nat(Nat.0) = Rat.0
        Rat.0 < Rat.from_nat(n)
        zero_lt_imp_pos(Rat.from_nat(n))
        Rat.from_nat(n).is_positive
        pos_ne_zero(Rat.from_nat(n))
        Rat.from_nat(n) != Rat.0
        real_from_rat_inverse(Rat.from_nat(n))
        Real.from_rat(Rat.from_nat(n).inverse) =
            Real.from_rat(Rat.from_nat(n)).inverse
        from_nat_is_from_rat(n)
        from_nat[Real](n) = Real.from_rat(Rat.from_nat(n))
        Real.1 / from_nat[Real](n) = from_nat[Real](n).inverse
        from_nat[Real](n).inverse = Real.from_rat(Rat.from_nat(n)).inverse
        real_unit_fraction(n) = Real.1 / from_nat[Real](n)
        real_unit_fraction(n) = Real.from_rat(Rat.from_nat(n)).inverse
        unit_fraction(n) = Rat.from_nat(n).inverse
        Real.from_rat(unit_fraction(n)) =
            Real.from_rat(Rat.from_nat(n).inverse)
        Real.from_rat(unit_fraction(n)) = real_unit_fraction(n)
    }
}

/// Unit fractions are antitone in positive denominators.
theorem unit_fraction_lte_of_lte(m: Nat, n: Nat) {
    Nat.0 < m and m <= n implies unit_fraction(n) <= unit_fraction(m)
} by {
    if Nat.0 < m and m <= n {
        lt_and_lte(Nat.0, m, n)
        Nat.0 < n
        nat_lt_imp_rat_lt(Nat.0, m)
        Rat.from_nat(Nat.0) < Rat.from_nat(m)
        Rat.from_nat(Nat.0) = Rat.0
        Rat.0 < Rat.from_nat(m)
        zero_lt_imp_pos(Rat.from_nat(m))
        Rat.from_nat(m).is_positive
        nat_lt_imp_rat_lt(Nat.0, n)
        Rat.from_nat(Nat.0) < Rat.from_nat(n)
        Rat.0 < Rat.from_nat(n)
        zero_lt_imp_pos(Rat.from_nat(n))
        Rat.from_nat(n).is_positive
        nat_lte_imp_rat_lte(m, n)
        Rat.from_nat(m) <= Rat.from_nat(n)
        mul_one_left(Rat.from_nat(m))
        Rat.1 * Rat.from_nat(m) = Rat.from_nat(m)
        mul_one_left(Rat.from_nat(n))
        Rat.1 * Rat.from_nat(n) = Rat.from_nat(n)
        Rat.1 * Rat.from_nat(m) <= Rat.1 * Rat.from_nat(n)
        cross_mul_lte(Rat.1, Rat.from_nat(n), Rat.1, Rat.from_nat(m))
        Rat.1 / Rat.from_nat(n) <= Rat.1 / Rat.from_nat(m)
        unit_fraction_eq_one_div(n)
        unit_fraction(n) = Rat.1 / Rat.from_nat(n)
        unit_fraction_eq_one_div(m)
        unit_fraction(m) = Rat.1 / Rat.from_nat(m)
        unit_fraction(n) <= unit_fraction(m)
    }
}

/// The previous unit fraction is bounded by twice the current unit fraction.
theorem unit_fraction_pred_lte_double(n: Nat) {
    Nat.1 < n implies unit_fraction(n - Nat.1) <= unit_fraction(n) + unit_fraction(n)
} by {
    if Nat.1 < n {
        Nat.0 < Nat.1
        Nat.1 <= n
        lt_and_lte(Nat.0, Nat.1, n)
        Nat.0 < n
        sub_pos(n, Nat.1)
        Nat.0 < n - Nat.1
        Nat.1 <= n - Nat.1
        add_sub(n, Nat.1)
        n - Nat.1 + Nat.1 = n
        n = n - Nat.1 + Nat.1
        n - Nat.1 + Nat.1 <= (n - Nat.1) + (n - Nat.1)
        mul_two_left(n - Nat.1)
        Nat.2 * (n - Nat.1) = (n - Nat.1) + (n - Nat.1)
        n <= Nat.2 * (n - Nat.1)
        nat_lt_imp_rat_lt(Nat.0, n - Nat.1)
        Rat.from_nat(Nat.0) < Rat.from_nat(n - Nat.1)
        Rat.from_nat(Nat.0) = Rat.0
        Rat.0 < Rat.from_nat(n - Nat.1)
        zero_lt_imp_pos(Rat.from_nat(n - Nat.1))
        Rat.from_nat(n - Nat.1).is_positive
        nat_lt_imp_rat_lt(Nat.0, n)
        Rat.from_nat(Nat.0) < Rat.from_nat(n)
        Rat.0 < Rat.from_nat(n)
        zero_lt_imp_pos(Rat.from_nat(n))
        Rat.from_nat(n).is_positive
        nat_lte_imp_rat_lte(n, Nat.2 * (n - Nat.1))
        Rat.from_nat(n) <= Rat.from_nat(Nat.2 * (n - Nat.1))
        from_nat_mul(Nat.2, n - Nat.1)
        Rat.from_nat(Nat.2) * Rat.from_nat(n - Nat.1) =
            Rat.from_nat(Nat.2 * (n - Nat.1))
        Rat.from_nat(Nat.2) = Rat.2
        Rat.2 * Rat.from_nat(n - Nat.1) =
            Rat.from_nat(Nat.2 * (n - Nat.1))
        Rat.1 * Rat.from_nat(n) = Rat.from_nat(n)
        Rat.1 * Rat.from_nat(n) <= Rat.2 * Rat.from_nat(n - Nat.1)
        cross_mul_lte(Rat.1, Rat.from_nat(n - Nat.1), Rat.2, Rat.from_nat(n))
        Rat.1 / Rat.from_nat(n - Nat.1) <= Rat.2 / Rat.from_nat(n)
        unit_fraction_eq_one_div(n - Nat.1)
        unit_fraction(n - Nat.1) = Rat.1 / Rat.from_nat(n - Nat.1)
        unit_fraction_eq_one_div(n)
        unit_fraction(n) = Rat.1 / Rat.from_nat(n)
        Rat.2 / Rat.from_nat(n) = Rat.2 * Rat.from_nat(n).inverse
        unit_fraction(n) = Rat.from_nat(n).inverse
        Rat.2 / Rat.from_nat(n) = Rat.2 * unit_fraction(n)
        times_two(unit_fraction(n))
        Rat.2 * unit_fraction(n) = unit_fraction(n) + unit_fraction(n)
        Rat.2 / Rat.from_nat(n) = unit_fraction(n) + unit_fraction(n)
        unit_fraction(n - Nat.1) <= unit_fraction(n) + unit_fraction(n)
    }
}

/// A positive rational below one has a unit-fraction subtraction step.
theorem unit_fraction_greedy_step_exists(q: Rat) {
    q.is_positive and q < Rat.1 implies exists(n: Nat) {
        unit_fraction_greedy_step(q, n)
    }
} by {
    if q.is_positive and q < Rat.1 {
        smaller_int_inverse(q)
        let d: Int satisfy {
            d.is_positive and Rat.from_int(d).inverse < q
        }
        positive_int_eq_from_nat_suc(d)
        let k: Nat satisfy {
            d = Int.from_nat(k.suc)
        }
        Nat.0 < k.suc
        Rat.from_int(d) = Rat.from_int(Int.from_nat(k.suc))
        Rat.from_int(Int.from_nat(k.suc)) = Rat.from_nat(k.suc)
        Rat.from_int(d).inverse = Rat.from_nat(k.suc).inverse
        unit_fraction(k.suc) = Rat.from_nat(k.suc).inverse
        unit_fraction(k.suc) < q
        unit_fraction_positive(k.suc)
        unit_fraction(k.suc).is_positive
        gt_minus_pos(q, unit_fraction(k.suc))
        q > q - unit_fraction(k.suc)
        q - unit_fraction(k.suc) < q
        unit_fraction_greedy_step(q, k.suc) =
            (Nat.0 < k.suc and unit_fraction(k.suc) < q and
                q - unit_fraction(k.suc) < q)
        unit_fraction_greedy_step(q, k.suc)
        exists(n: Nat) {
            unit_fraction_greedy_step(q, n)
        }
    }
}

/// A positive rational below one has a bounded unit-fraction subtraction step.
theorem unit_fraction_bounded_step_exists(q: Rat) {
    q.is_positive and q < Rat.1 implies exists(n: Nat) {
        unit_fraction_bounded_step(q, n)
    }
} by {
    if q.is_positive and q < Rat.1 {
        smaller_int_inverse(q)
        let d: Int satisfy {
            d.is_positive and Rat.from_int(d).inverse < q
        }
        positive_int_eq_from_nat_suc(d)
        let k: Nat satisfy {
            d = Int.from_nat(k.suc)
        }
        Nat.0 < k.suc
        Rat.from_int(d) = Rat.from_int(Int.from_nat(k.suc))
        Rat.from_int(Int.from_nat(k.suc)) = Rat.from_nat(k.suc)
        Rat.from_int(d).inverse = Rat.from_nat(k.suc).inverse
        unit_fraction(k.suc) = Rat.from_nat(k.suc).inverse
        unit_fraction(k.suc) < q
        unit_fraction(k.suc) <= q
        unit_fraction(k.suc) != q
        unit_fraction(k.suc) <= q =
            ((q - unit_fraction(k.suc)).is_positive or unit_fraction(k.suc) = q)
        (q - unit_fraction(k.suc)).is_positive or unit_fraction(k.suc) = q
        (q - unit_fraction(k.suc)).is_positive
        unit_fraction_positive(k.suc)
        unit_fraction(k.suc).is_positive
        gt_minus_pos(q, unit_fraction(k.suc))
        q > q - unit_fraction(k.suc)
        q - unit_fraction(k.suc) < q
        unit_fraction_bounded_step(q, k.suc) =
            (Nat.0 < k.suc and unit_fraction(k.suc) <= q and
                (q - unit_fraction(k.suc)).is_positive and
                q - unit_fraction(k.suc) < q)
        unit_fraction_bounded_step(q, k.suc)
        exists(n: Nat) {
            unit_fraction_bounded_step(q, n)
        }
    }
}

/// A bounded step has a positive denominator.
theorem unit_fraction_bounded_step_denominator_positive(q: Rat, n: Nat) {
    unit_fraction_bounded_step(q, n) implies Nat.0 < n
} by {
    if unit_fraction_bounded_step(q, n) {
        unit_fraction_bounded_step(q, n) =
            (Nat.0 < n and unit_fraction(n) <= q and
                (q - unit_fraction(n)).is_positive and
                q - unit_fraction(n) < q)
        Nat.0 < n
    }
}

lemma unit_fraction_bounded_step_lte(q: Rat, n: Nat) {
    unit_fraction_bounded_step(q, n) implies unit_fraction(n) <= q
} by {
    if unit_fraction_bounded_step(q, n) {
        unit_fraction_bounded_step(q, n) =
            (Nat.0 < n and unit_fraction(n) <= q and
                (q - unit_fraction(n)).is_positive and
                q - unit_fraction(n) < q)
        unit_fraction(n) <= q
    }
}

lemma unit_fraction_bounded_step_bounded_denominator(q: Rat, n: Nat) {
    unit_fraction_bounded_step(q, n) implies unit_fraction_bounded_denominator(q, n)
} by {
    if unit_fraction_bounded_step(q, n) {
        unit_fraction_bounded_step_denominator_positive(q, n)
        Nat.0 < n
        unit_fraction_bounded_step_lte(q, n)
        unit_fraction(n) <= q
        unit_fraction_bounded_denominator(q, n) =
            (Nat.0 < n and unit_fraction(n) <= q)
        unit_fraction_bounded_denominator(q, n)
    }
}

lemma unit_fraction_bounded_step_remainder_positive(q: Rat, n: Nat) {
    unit_fraction_bounded_step(q, n) implies (q - unit_fraction(n)).is_positive
} by {
    if unit_fraction_bounded_step(q, n) {
        unit_fraction_bounded_step(q, n) =
            (Nat.0 < n and unit_fraction(n) <= q and
                (q - unit_fraction(n)).is_positive and
                q - unit_fraction(n) < q)
        (q - unit_fraction(n)).is_positive
    }
}

/// The unreduced remainder denominator is positive.
theorem unit_fraction_remainder_raw_denom_positive(q: Rat, n: Nat) {
    Nat.0 < n implies unit_fraction_remainder_raw_denom(q, n).is_positive
} by {
    if Nat.0 < n {
        denom_positive(q)
        q.denom.is_positive
        from_nat_pos(n)
        Int.from_nat(n).is_positive
        mul_pos_pos(q.denom, Int.from_nat(n))
        (q.denom * Int.from_nat(n)).is_positive
        unit_fraction_remainder_raw_denom(q, n) =
            q.denom * Int.from_nat(n)
        unit_fraction_remainder_raw_denom(q, n).is_positive
    }
}

/// A bounded denominator is positive.
theorem unit_fraction_bounded_denominator_positive(q: Rat, n: Nat) {
    unit_fraction_bounded_denominator(q, n) implies Nat.0 < n
} by {
    if unit_fraction_bounded_denominator(q, n) {
        unit_fraction_bounded_denominator(q, n) =
            (Nat.0 < n and unit_fraction(n) <= q)
        Nat.0 < n
    }
}

/// A bounded denominator has a unit fraction below the rational.
theorem unit_fraction_bounded_denominator_lte(q: Rat, n: Nat) {
    unit_fraction_bounded_denominator(q, n) implies unit_fraction(n) <= q
} by {
    if unit_fraction_bounded_denominator(q, n) {
        unit_fraction_bounded_denominator(q, n) =
            (Nat.0 < n and unit_fraction(n) <= q)
        unit_fraction(n) <= q
    }
}

/// A bounded denominator gives a nonnegative unreduced remainder numerator.
theorem unit_fraction_bounded_denominator_raw_num_nonnegative(q: Rat, n: Nat) {
    unit_fraction_bounded_denominator(q, n) implies
        Int.0 <= unit_fraction_remainder_raw_num(q, n)
} by {
    if unit_fraction_bounded_denominator(q, n) {
        unit_fraction_bounded_denominator_positive(q, n)
        Nat.0 < n
        unit_fraction_bounded_denominator_lte(q, n)
        unit_fraction(n) <= q
        denom_positive(q)
        q.denom.is_positive
        from_nat_pos(n)
        Int.from_nat(n).is_positive
        nat_lt_imp_rat_lt(Nat.0, n)
        Rat.from_nat(Nat.0) < Rat.from_nat(n)
        Rat.from_nat(Nat.0) = Rat.0
        Rat.0 < Rat.from_nat(n)
        zero_lt_imp_pos(Rat.from_nat(n))
        Rat.from_nat(n).is_positive
        Rat.from_int(q.denom).is_positive
        lte_mul_pos(unit_fraction(n), q, Rat.from_int(q.denom))
        unit_fraction(n) * Rat.from_int(q.denom) <= q * Rat.from_int(q.denom)
        mul_denom(q)
        q * Rat.from_int(q.denom) = Rat.from_int(q.num)
        unit_fraction_eq_one_div(n)
        unit_fraction(n) = Rat.1 / Rat.from_nat(n)
        lte_mul_pos(unit_fraction(n) * Rat.from_int(q.denom),
            Rat.from_int(q.num), Rat.from_nat(n))
        unit_fraction(n) * Rat.from_int(q.denom) * Rat.from_nat(n) <= Rat.from_int(q.num) * Rat.from_nat(n)
        unit_fraction_mul_denominator(n)
        unit_fraction(n) * Rat.from_nat(n) = Rat.1
        mul_comm(Rat.from_int(q.denom), Rat.from_nat(n))
        Rat.from_int(q.denom) * Rat.from_nat(n) =
            Rat.from_nat(n) * Rat.from_int(q.denom)
        unit_fraction(n) * Rat.from_int(q.denom) * Rat.from_nat(n) =
            unit_fraction(n) * (Rat.from_int(q.denom) * Rat.from_nat(n))
        unit_fraction(n) * (Rat.from_int(q.denom) * Rat.from_nat(n)) =
            unit_fraction(n) * (Rat.from_nat(n) * Rat.from_int(q.denom))
        unit_fraction(n) * (Rat.from_nat(n) * Rat.from_int(q.denom)) =
            unit_fraction(n) * Rat.from_nat(n) * Rat.from_int(q.denom)
        unit_fraction(n) * Rat.from_int(q.denom) * Rat.from_nat(n) =
            unit_fraction(n) * Rat.from_nat(n) * Rat.from_int(q.denom)
        unit_fraction(n) * Rat.from_nat(n) * Rat.from_int(q.denom) =
            Rat.1 * Rat.from_int(q.denom)
        Rat.1 * Rat.from_int(q.denom) = Rat.from_int(q.denom)
        Rat.from_nat(n) = Rat.from_int(Int.from_nat(n))
        Rat.from_int(q.num) * Rat.from_nat(n) =
            Rat.from_int(q.num) * Rat.from_int(Int.from_nat(n))
        mul_int_eq_int_mul(q.num, Int.from_nat(n))
        Rat.from_int(q.num) * Rat.from_int(Int.from_nat(n)) =
            Rat.from_int(q.num * Int.from_nat(n))
        Rat.from_int(q.denom) <= Rat.from_int(q.num * Int.from_nat(n))
        from_int_lte_cancel(q.denom, q.num * Int.from_nat(n))
        q.denom <= q.num * Int.from_nat(n)
        sub_nonnegative_of_lte(q.denom, q.num * Int.from_nat(n))
        Int.0 <= q.num * Int.from_nat(n) - q.denom
        unit_fraction_remainder_raw_num(q, n) =
            q.num * Int.from_nat(n) - q.denom
        Int.0 <= unit_fraction_remainder_raw_num(q, n)
    }
}

/// The remainder after subtracting `1/n` is the raw fraction reduced.
theorem unit_fraction_remainder_eq_raw(q: Rat, n: Nat) {
    Nat.0 < n implies
        q - unit_fraction(n) =
            reduce(unit_fraction_remainder_raw_num(q, n),
                unit_fraction_remainder_raw_denom(q, n))
} by {
    if Nat.0 < n {
        denom_nonzero(q)
        q.denom != Int.0
        from_nat_pos(n)
        Int.from_nat(n).is_positive
        if Int.from_nat(n) = Int.0 {
            zero_not_pos
            false
        }
        Int.from_nat(n) != Int.0
        reduce_idempotent(q)
        reduce(q.num, q.denom) = q
        unit_fraction_eq_reduce(n)
        unit_fraction(n) = reduce(Int.1, Int.from_nat(n))
        reduce_neg_num(Int.1, Int.from_nat(n))
        reduce(-Int.1, Int.from_nat(n)) =
            -reduce(Int.1, Int.from_nat(n))
        -unit_fraction(n) = reduce(-Int.1, Int.from_nat(n))
        q - unit_fraction(n) = q + -unit_fraction(n)
        q + -unit_fraction(n) =
            reduce(q.num, q.denom) + reduce(-Int.1, Int.from_nat(n))
        add_reduced(q.num, q.denom, -Int.1, Int.from_nat(n))
        reduce(q.num, q.denom) + reduce(-Int.1, Int.from_nat(n)) =
            reduce(q.num * Int.from_nat(n) + q.denom * -Int.1,
                q.denom * Int.from_nat(n))
        q.denom * -Int.1 = -q.denom
        q.num * Int.from_nat(n) + q.denom * -Int.1 =
            q.num * Int.from_nat(n) - q.denom
        unit_fraction_remainder_raw_num(q, n) =
            q.num * Int.from_nat(n) - q.denom
        unit_fraction_remainder_raw_denom(q, n) =
            q.denom * Int.from_nat(n)
        q - unit_fraction(n) =
            reduce(unit_fraction_remainder_raw_num(q, n),
                unit_fraction_remainder_raw_denom(q, n))
    }
}

/// Every positive rational has a bounded unit-fraction denominator.
theorem unit_fraction_bounded_denominator_exists(q: Rat) {
    q.is_positive implies exists(n: Nat) {
        unit_fraction_bounded_denominator(q, n)
    }
} by {
    if q.is_positive {
        smaller_int_inverse(q)
        let d: Int satisfy {
            d.is_positive and Rat.from_int(d).inverse < q
        }
        positive_int_eq_from_nat_suc(d)
        let k: Nat satisfy {
            d = Int.from_nat(k.suc)
        }
        Nat.0 < k.suc
        Rat.from_int(d) = Rat.from_int(Int.from_nat(k.suc))
        Rat.from_int(Int.from_nat(k.suc)) = Rat.from_nat(k.suc)
        Rat.from_int(d).inverse = Rat.from_nat(k.suc).inverse
        unit_fraction(k.suc) = Rat.from_nat(k.suc).inverse
        unit_fraction(k.suc) < q
        unit_fraction(k.suc) <= q
        unit_fraction_bounded_denominator(q, k.suc) =
            (Nat.0 < k.suc and unit_fraction(k.suc) <= q)
        unit_fraction_bounded_denominator(q, k.suc)
        exists(n: Nat) {
            unit_fraction_bounded_denominator(q, n)
        }
    }
}

/// A canonical ceiling denominator exists for every positive rational.
theorem unit_fraction_ceiling_step_exists(q: Rat) {
    q.is_positive implies exists(n: Nat) {
        unit_fraction_ceiling_step(q, n)
    }
} by {
    if q.is_positive {
        unit_fraction_bounded_denominator_exists(q)
        let witness: Nat satisfy {
            unit_fraction_bounded_denominator(q, witness)
        }
        unit_fraction_bounded_denominator_for(q)(witness) =
            unit_fraction_bounded_denominator(q, witness)
        unit_fraction_bounded_denominator_for(q)(witness)
        has_min(unit_fraction_bounded_denominator_for(q), witness)
        let n: Nat satisfy {
            is_min(unit_fraction_bounded_denominator_for(q), n)
        }
        unit_fraction_ceiling_step(q, n) =
            is_min(unit_fraction_bounded_denominator_for(q), n)
        unit_fraction_ceiling_step(q, n)
        exists(result: Nat) {
            unit_fraction_ceiling_step(q, result)
        }
    }
}

/// A canonical ceiling denominator is a bounded denominator.
theorem unit_fraction_ceiling_step_bounded_denominator(q: Rat, n: Nat) {
    unit_fraction_ceiling_step(q, n) implies
        unit_fraction_bounded_denominator(q, n)
} by {
    if unit_fraction_ceiling_step(q, n) {
        unit_fraction_ceiling_step(q, n) =
            is_min(unit_fraction_bounded_denominator_for(q), n)
        is_min_apply(unit_fraction_bounded_denominator_for(q), n)
        unit_fraction_bounded_denominator_for(q)(n)
        unit_fraction_bounded_denominator_for(q)(n) =
            unit_fraction_bounded_denominator(q, n)
        unit_fraction_bounded_denominator(q, n)
    }
}

/// A canonical ceiling denominator is positive.
theorem unit_fraction_ceiling_step_denominator_positive(q: Rat, n: Nat) {
    unit_fraction_ceiling_step(q, n) implies Nat.0 < n
} by {
    if unit_fraction_ceiling_step(q, n) {
        unit_fraction_ceiling_step_bounded_denominator(q, n)
        unit_fraction_bounded_denominator(q, n)
        unit_fraction_bounded_denominator_positive(q, n)
        Nat.0 < n
    }
}

/// A canonical ceiling denominator has unit fraction below the rational.
theorem unit_fraction_ceiling_step_lte(q: Rat, n: Nat) {
    unit_fraction_ceiling_step(q, n) implies unit_fraction(n) <= q
} by {
    if unit_fraction_ceiling_step(q, n) {
        unit_fraction_ceiling_step_bounded_denominator(q, n)
        unit_fraction_bounded_denominator(q, n)
        unit_fraction_bounded_denominator_lte(q, n)
        unit_fraction(n) <= q
    }
}

/// A canonical ceiling denominator is no larger than any bounded denominator.
theorem unit_fraction_ceiling_step_minimal(q: Rat, n: Nat, m: Nat) {
    unit_fraction_ceiling_step(q, n) and unit_fraction_bounded_denominator(q, m)
        implies n <= m
} by {
    if unit_fraction_ceiling_step(q, n) and unit_fraction_bounded_denominator(q, m) {
        unit_fraction_ceiling_step(q, n) =
            is_min(unit_fraction_bounded_denominator_for(q), n)
        is_min_false_below(unit_fraction_bounded_denominator_for(q), n)
        false_below(unit_fraction_bounded_denominator_for(q), n)
        if m < n {
            false_below_apply(unit_fraction_bounded_denominator_for(q), n, m)
            not unit_fraction_bounded_denominator_for(q)(m)
            unit_fraction_bounded_denominator_for(q)(m) =
                unit_fraction_bounded_denominator(q, m)
            false
        }
        not m < n
        lt_or_lte(m, n)
        n <= m
    }
}

/// A canonical ceiling step either finishes exactly or gives a proper bounded step.
theorem unit_fraction_ceiling_step_terminal_or_bounded(q: Rat, n: Nat) {
    unit_fraction_ceiling_step(q, n) implies
        q = unit_fraction(n) or unit_fraction_bounded_step(q, n)
} by {
    if unit_fraction_ceiling_step(q, n) {
        unit_fraction_ceiling_step_denominator_positive(q, n)
        Nat.0 < n
        unit_fraction_ceiling_step_lte(q, n)
        unit_fraction(n) <= q
        if unit_fraction(n) = q {
            q = unit_fraction(n)
        } else {
            unit_fraction(n) != q
            unit_fraction(n) <= q =
                ((q - unit_fraction(n)).is_positive or unit_fraction(n) = q)
            (q - unit_fraction(n)).is_positive
            unit_fraction_positive(n)
            unit_fraction(n).is_positive
            gt_minus_pos(q, unit_fraction(n))
            q > q - unit_fraction(n)
            q - unit_fraction(n) < q
            unit_fraction_bounded_step(q, n) =
                (Nat.0 < n and unit_fraction(n) <= q and
                    (q - unit_fraction(n)).is_positive and
                    q - unit_fraction(n) < q)
            unit_fraction_bounded_step(q, n)
        }
    }
}

/// For a nontrivial canonical ceiling denominator, the previous denominator is too small.
theorem unit_fraction_ceiling_step_pred_lt(q: Rat, n: Nat) {
    unit_fraction_ceiling_step(q, n) and Nat.1 < n
        implies q < unit_fraction(n - Nat.1)
} by {
    if unit_fraction_ceiling_step(q, n) and Nat.1 < n {
        Nat.0 < n
        sub_one_lt(n)
        n - Nat.1 < n
        sub_pos(n, Nat.1)
        Nat.0 < n - Nat.1
        if unit_fraction(n - Nat.1) <= q {
            unit_fraction_bounded_denominator(q, n - Nat.1) =
                (Nat.0 < n - Nat.1 and unit_fraction(n - Nat.1) <= q)
            unit_fraction_bounded_denominator(q, n - Nat.1)
            unit_fraction_ceiling_step_minimal(q, n, n - Nat.1)
            n <= n - Nat.1
            lte_imp_not_lt(n, n - Nat.1)
            false
        }
        not unit_fraction(n - Nat.1) <= q
        rat_total(q, unit_fraction(n - Nat.1))
        q <= unit_fraction(n - Nat.1)
        if q = unit_fraction(n - Nat.1) {
            unit_fraction(n - Nat.1) <= q
            false
        }
        q != unit_fraction(n - Nat.1)
        q < unit_fraction(n - Nat.1) =
            (q <= unit_fraction(n - Nat.1) and
                q != unit_fraction(n - Nat.1))
        q < unit_fraction(n - Nat.1)
    }
}

/// A canonical ceiling denominator for a rational below one is greater than one.
theorem unit_fraction_ceiling_step_gt_one_of_lt_one(q: Rat, n: Nat) {
    q < Rat.1 and unit_fraction_ceiling_step(q, n) implies Nat.1 < n
} by {
    if q < Rat.1 and unit_fraction_ceiling_step(q, n) {
        unit_fraction_ceiling_step_denominator_positive(q, n)
        Nat.0 < n
        Nat.1 <= n
        unit_fraction_ceiling_step_lte(q, n)
        unit_fraction(n) <= q
        lte_lt_trans(unit_fraction(n), q, Rat.1)
        unit_fraction(n) < Rat.1
        lt_or_lte(Nat.1, n)
        if n <= Nat.1 {
            lte_antisymm(Nat.1, n)
            n = Nat.1
            unit_fraction_one
            unit_fraction(n) = Rat.1
            false
        }
        Nat.1 < n
    }
}

/// A canonical ceiling denominator below one gives a smaller unreduced numerator.
theorem unit_fraction_ceiling_step_raw_num_lt_num(q: Rat, n: Nat) {
    q < Rat.1 and unit_fraction_ceiling_step(q, n)
        implies unit_fraction_remainder_raw_num(q, n) < q.num
} by {
    if q < Rat.1 and unit_fraction_ceiling_step(q, n) {
        unit_fraction_ceiling_step_gt_one_of_lt_one(q, n)
        Nat.1 < n
        let m = n - Nat.1
        m = n - Nat.1
        sub_pos(n, Nat.1)
        Nat.0 < n - Nat.1
        Nat.0 < m
        unit_fraction_ceiling_step_pred_lt(q, n)
        q < unit_fraction(m)
        denom_positive(q)
        q.denom.is_positive
        Rat.from_int(q.denom).is_positive
        from_nat_pos(m)
        Int.from_nat(m).is_positive
        nat_lt_imp_rat_lt(Nat.0, m)
        Rat.from_nat(Nat.0) < Rat.from_nat(m)
        Rat.from_nat(Nat.0) = Rat.0
        Rat.0 < Rat.from_nat(m)
        zero_lt_imp_pos(Rat.from_nat(m))
        Rat.from_nat(m).is_positive
        lt_mul_pos(q, unit_fraction(m), Rat.from_int(q.denom))
        q * Rat.from_int(q.denom) < unit_fraction(m) * Rat.from_int(q.denom)
        mul_denom(q)
        q * Rat.from_int(q.denom) = Rat.from_int(q.num)
        lt_mul_pos(Rat.from_int(q.num),
            unit_fraction(m) * Rat.from_int(q.denom), Rat.from_nat(m))
        Rat.from_int(q.num) * Rat.from_nat(m) < unit_fraction(m) * Rat.from_int(q.denom) * Rat.from_nat(m)
        unit_fraction_mul_denominator(m)
        unit_fraction(m) * Rat.from_nat(m) = Rat.1
        mul_comm(Rat.from_int(q.denom), Rat.from_nat(m))
        Rat.from_int(q.denom) * Rat.from_nat(m) =
            Rat.from_nat(m) * Rat.from_int(q.denom)
        unit_fraction(m) * Rat.from_int(q.denom) * Rat.from_nat(m) =
            unit_fraction(m) * (Rat.from_int(q.denom) * Rat.from_nat(m))
        unit_fraction(m) * (Rat.from_int(q.denom) * Rat.from_nat(m)) =
            unit_fraction(m) * (Rat.from_nat(m) * Rat.from_int(q.denom))
        unit_fraction(m) * (Rat.from_nat(m) * Rat.from_int(q.denom)) =
            unit_fraction(m) * Rat.from_nat(m) * Rat.from_int(q.denom)
        unit_fraction(m) * Rat.from_int(q.denom) * Rat.from_nat(m) =
            unit_fraction(m) * Rat.from_nat(m) * Rat.from_int(q.denom)
        unit_fraction(m) * Rat.from_nat(m) * Rat.from_int(q.denom) =
            Rat.1 * Rat.from_int(q.denom)
        Rat.1 * Rat.from_int(q.denom) = Rat.from_int(q.denom)
        Rat.from_nat(m) = Rat.from_int(Int.from_nat(m))
        Rat.from_int(q.num) * Rat.from_nat(m) =
            Rat.from_int(q.num) * Rat.from_int(Int.from_nat(m))
        mul_int_eq_int_mul(q.num, Int.from_nat(m))
        Rat.from_int(q.num) * Rat.from_int(Int.from_nat(m)) =
            Rat.from_int(q.num * Int.from_nat(m))
        Rat.from_int(q.num * Int.from_nat(m)) < Rat.from_int(q.denom)
        from_int_lt_cancel(q.num * Int.from_nat(m), q.denom)
        q.num * Int.from_nat(m) < q.denom
        Nat.1 <= n
        add_sub(n, Nat.1)
        n - Nat.1 + Nat.1 = n
        m + Nat.1 = n
        Int.from_nat(n) = Int.from_nat(m + Nat.1)
        Int.from_nat(m + Nat.1) = Int.from_nat(m) + Int.from_nat(Nat.1)
        Int.from_nat(Nat.1) = Int.1
        q.num * Int.from_nat(n) = q.num * (Int.from_nat(m) + Int.1)
        q.num * (Int.from_nat(m) + Int.1) =
            q.num * Int.from_nat(m) + q.num
        q.num * Int.from_nat(n) =
            q.num * Int.from_nat(m) + q.num
        add_sub_lt_right_of_lt(q.num * Int.from_nat(m), q.denom, q.num)
        q.num * Int.from_nat(m) + q.num - q.denom < q.num
        q.num * Int.from_nat(n) - q.denom < q.num
        unit_fraction_remainder_raw_num(q, n) =
            q.num * Int.from_nat(n) - q.denom
        unit_fraction_remainder_raw_num(q, n) < q.num
    }
}

/// A positive canonical remainder below one has a smaller reduced numerator.
theorem unit_fraction_ceiling_step_positive_remainder_num_lt_num(q: Rat, n: Nat) {
    q < Rat.1 and unit_fraction_ceiling_step(q, n) and
        (q - unit_fraction(n)).is_positive
        implies (q - unit_fraction(n)).num < q.num
} by {
    if q < Rat.1 and unit_fraction_ceiling_step(q, n) and
        (q - unit_fraction(n)).is_positive {
        unit_fraction_ceiling_step_denominator_positive(q, n)
        Nat.0 < n
        let raw_num = unit_fraction_remainder_raw_num(q, n)
        let raw_denom = unit_fraction_remainder_raw_denom(q, n)
        unit_fraction_remainder_eq_raw(q, n)
        q - unit_fraction(n) = reduce(raw_num, raw_denom)
        reduce(raw_num, raw_denom).is_positive
        unit_fraction_remainder_raw_denom_positive(q, n)
        raw_denom.is_positive
        reduce_num_lte_raw_of_positive(raw_num, raw_denom)
        reduce(raw_num, raw_denom).num <= raw_num
        (q - unit_fraction(n)).num = reduce(raw_num, raw_denom).num
        (q - unit_fraction(n)).num <= raw_num
        unit_fraction_ceiling_step_raw_num_lt_num(q, n)
        raw_num < q.num
        lt_of_lte_of_lt[Int]((q - unit_fraction(n)).num, raw_num, q.num)
        (q - unit_fraction(n)).num < q.num
    }
}

/// A nonterminal canonical ceiling step below one has a smaller reduced numerator.
theorem unit_fraction_ceiling_step_remainder_num_lt_num(q: Rat, n: Nat) {
    q < Rat.1 and unit_fraction_ceiling_step(q, n) and q != unit_fraction(n)
        implies (q - unit_fraction(n)).num < q.num
} by {
    if q < Rat.1 and unit_fraction_ceiling_step(q, n) and q != unit_fraction(n) {
        unit_fraction_ceiling_step_lte(q, n)
        unit_fraction(n) <= q
        if unit_fraction(n) = q {
            q = unit_fraction(n)
            false
        } else {
            unit_fraction(n) != q
            unit_fraction(n) <= q =
                ((q - unit_fraction(n)).is_positive or unit_fraction(n) = q)
            (q - unit_fraction(n)).is_positive or unit_fraction(n) = q
            (q - unit_fraction(n)).is_positive
            unit_fraction_ceiling_step_positive_remainder_num_lt_num(q, n)
            (q - unit_fraction(n)).num < q.num
        }
    }
}

/// A nonterminal canonical ceiling step leaves a positive remainder.
theorem unit_fraction_ceiling_step_remainder_positive(q: Rat, n: Nat) {
    unit_fraction_ceiling_step(q, n) and q != unit_fraction(n)
        implies (q - unit_fraction(n)).is_positive
} by {
    if unit_fraction_ceiling_step(q, n) and q != unit_fraction(n) {
        unit_fraction_ceiling_step_lte(q, n)
        unit_fraction(n) <= q
        if unit_fraction(n) = q {
            q = unit_fraction(n)
            false
        } else {
            unit_fraction(n) != q
            unit_fraction(n) <= q =
                ((q - unit_fraction(n)).is_positive or unit_fraction(n) = q)
            (q - unit_fraction(n)).is_positive or unit_fraction(n) = q
            (q - unit_fraction(n)).is_positive
        }
    }
}

/// A nonterminal canonical ceiling step leaves a smaller remainder.
theorem unit_fraction_ceiling_step_remainder_lt_self(q: Rat, n: Nat) {
    unit_fraction_ceiling_step(q, n) and q != unit_fraction(n)
        implies q - unit_fraction(n) < q
} by {
    if unit_fraction_ceiling_step(q, n) and q != unit_fraction(n) {
        unit_fraction_ceiling_step_denominator_positive(q, n)
        Nat.0 < n
        unit_fraction_positive(n)
        unit_fraction(n).is_positive
        gt_minus_pos(q, unit_fraction(n))
        q > q - unit_fraction(n)
        q - unit_fraction(n) < q
    }
}

/// A nonterminal canonical ceiling step below one leaves a remainder below one.
theorem unit_fraction_ceiling_step_remainder_lt_one(q: Rat, n: Nat) {
    q < Rat.1 and unit_fraction_ceiling_step(q, n) and q != unit_fraction(n)
        implies q - unit_fraction(n) < Rat.1
} by {
    if q < Rat.1 and unit_fraction_ceiling_step(q, n) and q != unit_fraction(n) {
        unit_fraction_ceiling_step_remainder_lt_self(q, n)
        q - unit_fraction(n) < q
        lt_trans(q - unit_fraction(n), q, Rat.1)
        q - unit_fraction(n) < Rat.1
    }
}

/// A canonical ceiling step below one leaves a remainder below the chosen unit
/// fraction.
theorem unit_fraction_ceiling_step_remainder_lt_unit(q: Rat, n: Nat) {
    q < Rat.1 and unit_fraction_ceiling_step(q, n)
        implies q - unit_fraction(n) < unit_fraction(n)
} by {
    if q < Rat.1 and unit_fraction_ceiling_step(q, n) {
        unit_fraction_ceiling_step_gt_one_of_lt_one(q, n)
        Nat.1 < n
        unit_fraction_ceiling_step_pred_lt(q, n)
        q < unit_fraction(n - Nat.1)
        unit_fraction_pred_lte_double(n)
        unit_fraction(n - Nat.1) <= unit_fraction(n) + unit_fraction(n)
        lt_lte_trans(q, unit_fraction(n - Nat.1),
            unit_fraction(n) + unit_fraction(n))
        q < unit_fraction(n) + unit_fraction(n)
        q - unit_fraction(n) + unit_fraction(n) = q
        q - unit_fraction(n) + unit_fraction(n) < unit_fraction(n) + unit_fraction(n)
        lt_cancel_add_right(q - unit_fraction(n), unit_fraction(n),
            unit_fraction(n))
        q - unit_fraction(n) < unit_fraction(n)
    }
}

/// The next canonical greedy denominator after a canonical step is larger.
theorem unit_fraction_ceiling_step_next_denominator_gt(q: Rat, n: Nat, m: Nat) {
    q < Rat.1 and unit_fraction_ceiling_step(q, n) and
        unit_fraction_ceiling_step(q - unit_fraction(n), m)
        implies n < m
} by {
    if q < Rat.1 {
        if unit_fraction_ceiling_step(q, n) {
            if unit_fraction_ceiling_step(q - unit_fraction(n), m) {
                q < Rat.1 and unit_fraction_ceiling_step(q, n)
                unit_fraction_ceiling_step_remainder_lt_unit(q, n)
                q - unit_fraction(n) < unit_fraction(n)
                unit_fraction_ceiling_step_denominator_positive(
                    q - unit_fraction(n), m)
                Nat.0 < m
                unit_fraction_ceiling_step_lte(q - unit_fraction(n), m)
                unit_fraction(m) <= q - unit_fraction(n)
                if m <= n {
                    unit_fraction_lte_of_lte(m, n)
                    unit_fraction(n) <= unit_fraction(m)
                    lte_trans(unit_fraction(n), unit_fraction(m),
                        q - unit_fraction(n))
                    unit_fraction(n) <= q - unit_fraction(n)
                    lte_lt_trans(unit_fraction(n), q - unit_fraction(n),
                        unit_fraction(n))
                    unit_fraction(n) < unit_fraction(n)
                    not_lt_self(unit_fraction(n))
                    false
                }
                not m <= n
                lt_or_lte(n, m)
                n < m
            }
        }
    }
}

/// The next canonical greedy denominator is lower-bounded by the successor of
/// the current denominator.
theorem unit_fraction_ceiling_step_next_denominator_lower_bound(q: Rat, n: Nat,
    m: Nat) {
    q < Rat.1 and unit_fraction_ceiling_step(q, n) and
        unit_fraction_ceiling_step(q - unit_fraction(n), m)
        implies n.suc <= m
} by {
    if q < Rat.1 and unit_fraction_ceiling_step(q, n) and
        unit_fraction_ceiling_step(q - unit_fraction(n), m) {
        unit_fraction_ceiling_step_next_denominator_gt(q, n, m)
        n < m
        lt_imp_lte_suc(n, m)
        n.suc <= m
    }
}
/// The zero raw numerator is exactly a zero reduced remainder.
theorem unit_fraction_remainder_zero_of_raw_num_zero(q: Rat, n: Nat) {
    Nat.0 < n and unit_fraction_remainder_raw_num(q, n) = Int.0 implies
        q - unit_fraction(n) = Rat.0
} by {
    if Nat.0 < n and unit_fraction_remainder_raw_num(q, n) = Int.0 {
        unit_fraction_remainder_eq_raw(q, n)
        q - unit_fraction(n) =
            reduce(unit_fraction_remainder_raw_num(q, n),
                unit_fraction_remainder_raw_denom(q, n))
        q - unit_fraction(n) = reduce(Int.0,
            unit_fraction_remainder_raw_denom(q, n))
        reduce_zero_num(unit_fraction_remainder_raw_denom(q, n))
        reduce(Int.0, unit_fraction_remainder_raw_denom(q, n)) = Rat.0
        q - unit_fraction(n) = Rat.0
    }
}

/// A positive bounded-step remainder has positive raw numerator.
theorem unit_fraction_bounded_step_raw_num_positive(q: Rat, n: Nat) {
    unit_fraction_bounded_step(q, n) implies
        unit_fraction_remainder_raw_num(q, n).is_positive
} by {
    if unit_fraction_bounded_step(q, n) {
        unit_fraction_bounded_step_denominator_positive(q, n)
        Nat.0 < n
        unit_fraction_bounded_step_bounded_denominator(q, n)
        unit_fraction_bounded_denominator(q, n)
        unit_fraction_bounded_denominator_raw_num_nonnegative(q, n)
        Int.0 <= unit_fraction_remainder_raw_num(q, n)
        unit_fraction_bounded_step_remainder_positive(q, n)
        (q - unit_fraction(n)).is_positive
        if unit_fraction_remainder_raw_num(q, n) = Int.0 {
            unit_fraction_remainder_zero_of_raw_num_zero(q, n)
            q - unit_fraction(n) = Rat.0
            Rat.0.is_positive
            false
        }
        unit_fraction_remainder_raw_num(q, n) != Int.0
        nonzero_pos_or_neg(unit_fraction_remainder_raw_num(q, n))
        unit_fraction_remainder_raw_num(q, n).is_positive or
            unit_fraction_remainder_raw_num(q, n).is_negative
        if unit_fraction_remainder_raw_num(q, n).is_negative {
            Int.0 <= unit_fraction_remainder_raw_num(q, n)
            neg_lt_nonneg(unit_fraction_remainder_raw_num(q, n), Int.0)
            unit_fraction_remainder_raw_num(q, n) < Int.0
            lte_and_lt(Int.0, unit_fraction_remainder_raw_num(q, n), Int.0)
            Int.0 < Int.0
            lt_not_ref(Int.0)
            false
        }
        unit_fraction_remainder_raw_num(q, n).is_positive
    }
}

lemma positive_int_lt_imp_abs_lt(a: Int, b: Int) {
    a.is_positive and b.is_positive and a < b implies abs(a) < abs(b)
} by {
    if a.is_positive and b.is_positive and a < b {
        member_abs_pos(a)
        a.abs = a
        Int.from_nat(abs(a)) = a
        member_abs_pos(b)
        b.abs = b
        Int.from_nat(abs(b)) = b
        Int.from_nat(abs(a)) < Int.from_nat(abs(b))
        Int.from_nat(abs(a)) <= Int.from_nat(abs(b))
        lte_nonnegative_ints_implies_nats(abs(a), abs(b))
        abs(a) <= abs(b)
        if abs(a) = abs(b) {
            Int.from_nat(abs(a)) = Int.from_nat(abs(b))
            a = b
            false
        }
        abs(a) != abs(b)
        abs(a) < abs(b)
    }
}

/// A canonical bounded step below one gives strict descent of the raw numerator.
theorem unit_fraction_ceiling_step_raw_num_descends(q: Rat, n: Nat) {
    q.is_positive and q < Rat.1 and unit_fraction_ceiling_step(q, n) and
        unit_fraction_bounded_step(q, n)
        implies unit_fraction_raw_num_descends(q, n)
} by {
    if q.is_positive and q < Rat.1 and unit_fraction_ceiling_step(q, n) and
        unit_fraction_bounded_step(q, n) {
        unit_fraction_bounded_step_raw_num_positive(q, n)
        unit_fraction_remainder_raw_num(q, n).is_positive
        unit_fraction_ceiling_step_raw_num_lt_num(q, n)
        unit_fraction_remainder_raw_num(q, n) < q.num
        unit_fraction_raw_num_descends(q, n) =
            (unit_fraction_remainder_raw_num(q, n).is_positive and
                unit_fraction_remainder_raw_num(q, n) < q.num)
        unit_fraction_raw_num_descends(q, n)
    }
}

/// A canonical bounded step below one strictly decreases the natural numerator measure.
theorem unit_fraction_ceiling_step_raw_num_measure_lt(q: Rat, n: Nat) {
    q.is_positive and q < Rat.1 and unit_fraction_ceiling_step(q, n) and
        unit_fraction_bounded_step(q, n)
        implies unit_fraction_remainder_raw_num_measure(q, n) <
            unit_fraction_num_measure(q)
} by {
    if q.is_positive and q < Rat.1 and unit_fraction_ceiling_step(q, n) and
        unit_fraction_bounded_step(q, n) {
        unit_fraction_ceiling_step_raw_num_descends(q, n)
        unit_fraction_raw_num_descends(q, n)
        unit_fraction_raw_num_descends(q, n) =
            (unit_fraction_remainder_raw_num(q, n).is_positive and
                unit_fraction_remainder_raw_num(q, n) < q.num)
        unit_fraction_remainder_raw_num(q, n).is_positive
        unit_fraction_remainder_raw_num(q, n) < q.num
        q.is_positive = q.num.is_positive
        q.num.is_positive
        positive_int_lt_imp_abs_lt(unit_fraction_remainder_raw_num(q, n), q.num)
        abs(unit_fraction_remainder_raw_num(q, n)) < abs(q.num)
        unit_fraction_remainder_raw_num_measure(q, n) =
            abs(unit_fraction_remainder_raw_num(q, n))
        unit_fraction_num_measure(q) = abs(q.num)
        unit_fraction_remainder_raw_num_measure(q, n) < unit_fraction_num_measure(q)
    }
}

/// The empty unit-fraction sum is zero.
theorem unit_fraction_sum_nil {
    unit_fraction_sum(List.nil[Nat]) = Rat.0
} by {
    map[Nat, Rat](List.nil[Nat], unit_fraction) = List.nil[Rat]
    sum[Rat](List.nil[Rat]) = Rat.0
}

/// The empty real unit-fraction sum is zero.
theorem real_unit_fraction_sum_nil {
    real_unit_fraction_sum(List.nil[Nat]) = Real.0
} by {
    map[Nat, Real](List.nil[Nat], real_unit_fraction) = List.nil[Real]
    sum[Real](List.nil[Real]) = Real.0
}

/// Consing a denominator adds its unit fraction to the front of the sum.
theorem unit_fraction_sum_cons(n: Nat, denominators: List[Nat]) {
    unit_fraction_sum(List.cons(n, denominators)) =
        unit_fraction(n) + unit_fraction_sum(denominators)
} by {
    map[Nat, Rat](List.cons(n, denominators), unit_fraction) =
        List.cons(unit_fraction(n), map(denominators, unit_fraction))
    sum[Rat](List.cons(unit_fraction(n), map(denominators, unit_fraction))) =
        unit_fraction(n) + sum(map(denominators, unit_fraction))
}

/// Consing a denominator adds its real unit fraction to the front of the sum.
theorem real_unit_fraction_sum_cons(n: Nat, denominators: List[Nat]) {
    real_unit_fraction_sum(List.cons(n, denominators)) =
        real_unit_fraction(n) + real_unit_fraction_sum(denominators)
} by {
    map[Nat, Real](List.cons(n, denominators), real_unit_fraction) =
        List.cons(real_unit_fraction(n), map(denominators, real_unit_fraction))
    sum[Real](List.cons(real_unit_fraction(n), map(denominators, real_unit_fraction))) =
        real_unit_fraction(n) + sum(map(denominators, real_unit_fraction))
}

/// A singleton unit-fraction sum is that unit fraction.
theorem unit_fraction_sum_singleton(n: Nat) {
    unit_fraction_sum(List.singleton(n)) = unit_fraction(n)
} by {
    unit_fraction_sum_cons(n, List.nil[Nat])
    unit_fraction_sum_nil
    unit_fraction(n) + Rat.0 = unit_fraction(n)
}

/// The unit-fraction sum of a concatenation splits as a sum.
theorem unit_fraction_sum_append(left: List[Nat], right: List[Nat]) {
    unit_fraction_sum(left + right) = unit_fraction_sum(left) + unit_fraction_sum(right)
} by {
    map_add[Nat, Rat](left, right, unit_fraction)
    map(left + right, unit_fraction) = map(left, unit_fraction) + map(right, unit_fraction)
    sum_add[Rat](map(left, unit_fraction), map(right, unit_fraction))
    sum(map(left, unit_fraction) + map(right, unit_fraction)) =
        sum(map(left, unit_fraction)) + sum(map(right, unit_fraction))
}

/// The empty list has positive denominators.
theorem positive_denominator_list_nil {
    positive_denominator_list(List.nil[Nat])
} by {
    positive_denominator_list(List.nil[Nat]) = true
}

/// Consing a positive denominator onto a positive denominator list preserves positivity.
theorem positive_denominator_list_cons(n: Nat, denominators: List[Nat]) {
    Nat.0 < n and positive_denominator_list(denominators)
        implies positive_denominator_list(List.cons(n, denominators))
} by {
    if Nat.0 < n and positive_denominator_list(denominators) {
        positive_denominator_list(List.cons(n, denominators)) =
            (Nat.0 < n and positive_denominator_list(denominators))
    }
}

/// A nonempty positive denominator list has a positive head.
theorem positive_denominator_list_head(n: Nat, denominators: List[Nat]) {
    positive_denominator_list(List.cons(n, denominators)) implies Nat.0 < n
} by {
    if positive_denominator_list(List.cons(n, denominators)) {
        Nat.0 < n
    }
}

/// The tail of a nonempty positive denominator list has positive denominators.
theorem positive_denominator_list_tail(n: Nat, denominators: List[Nat]) {
    positive_denominator_list(List.cons(n, denominators))
        implies positive_denominator_list(denominators)
} by {
    if positive_denominator_list(List.cons(n, denominators)) {
        positive_denominator_list(denominators)
    }
}

/// Inductive predicate for preserving positivity under append.
define positive_denominator_list_append_pred(right: List[Nat], left: List[Nat]) -> Bool {
    positive_denominator_list(left) and positive_denominator_list(right)
        implies positive_denominator_list(left + right)
}

/// Positivity append preservation holds for every left list.
theorem positive_denominator_list_append_all(right: List[Nat]) {
    forall(left: List[Nat]) {
        positive_denominator_list_append_pred(right, left)
    }
} by {
    if positive_denominator_list(List.nil[Nat]) and positive_denominator_list(right) {
        List.nil[Nat] + right = right
        positive_denominator_list(List.nil[Nat] + right)
    }
    positive_denominator_list_append_pred(right, List.nil[Nat])

    forall(head: Nat, tail: List[Nat]) {
        if positive_denominator_list_append_pred(right, tail) {
            if positive_denominator_list(List.cons(head, tail)) and positive_denominator_list(right) {
                positive_denominator_list_head(head, tail)
                Nat.0 < head
                positive_denominator_list_tail(head, tail)
                positive_denominator_list(tail)
                positive_denominator_list_append_pred(right, tail) =
                    (positive_denominator_list(tail) and positive_denominator_list(right)
                        implies positive_denominator_list(tail + right))
                positive_denominator_list(tail + right)
                positive_denominator_list_cons(head, tail + right)
                positive_denominator_list(List.cons(head, tail + right))
                List.cons(head, tail) + right = List.cons(head, tail + right)
                positive_denominator_list(List.cons(head, tail) + right)
            }
            positive_denominator_list_append_pred(right, List.cons(head, tail)) =
                (positive_denominator_list(List.cons(head, tail)) and positive_denominator_list(right)
                    implies positive_denominator_list(List.cons(head, tail) + right))
            positive_denominator_list_append_pred(right, List.cons(head, tail))
        }
    }
    forall(head: Nat, tail: List[Nat]) {
        positive_denominator_list_append_pred(right, tail) implies positive_denominator_list_append_pred(right, List.cons(head, tail))
    }
    List.induction(function(items: List[Nat]) { positive_denominator_list_append_pred(right, items) })
}

/// Positive denominator lists remain positive after append.
theorem positive_denominator_list_append(left: List[Nat], right: List[Nat]) {
    positive_denominator_list(left) and positive_denominator_list(right)
        implies positive_denominator_list(left + right)
} by {
    positive_denominator_list_append_all(right)
    positive_denominator_list_append_pred(right, left)
    positive_denominator_list_append_pred(right, left) =
        (positive_denominator_list(left) and positive_denominator_list(right)
            implies positive_denominator_list(left + right))
}

/// The empty list satisfies every lower bound.
theorem denominator_list_lower_bound_nil(bound: Nat) {
    denominator_list_lower_bound(bound, List.nil[Nat])
} by {
    lower_bound_nil[Nat](bound)
}

/// The empty list satisfies every upper bound.
theorem denominator_list_upper_bound_nil(bound: Nat) {
    denominator_list_upper_bound(bound, List.nil[Nat])
} by {
    upper_bound_nil[Nat](bound)
}

/// Consing a denominator above the lower bound preserves the lower bound.
theorem denominator_list_lower_bound_cons(bound: Nat, n: Nat, denominators: List[Nat]) {
    bound <= n and denominator_list_lower_bound(bound, denominators)
        implies denominator_list_lower_bound(bound, List.cons(n, denominators))
} by {
    if bound <= n and denominator_list_lower_bound(bound, denominators) {
        lower_bound_cons_iff[Nat](n, denominators, bound)
        is_lower_bound(List.cons(n, denominators), bound) =
            (bound <= n and is_lower_bound(denominators, bound))
        denominator_list_lower_bound(bound, List.cons(n, denominators))
    }
}

/// Consing a denominator below the upper bound preserves the upper bound.
theorem denominator_list_upper_bound_cons(bound: Nat, n: Nat, denominators: List[Nat]) {
    n <= bound and denominator_list_upper_bound(bound, denominators)
        implies denominator_list_upper_bound(bound, List.cons(n, denominators))
} by {
    if n <= bound and denominator_list_upper_bound(bound, denominators) {
        upper_bound_cons_iff[Nat](n, denominators, bound)
        is_upper_bound(List.cons(n, denominators), bound) =
            (n <= bound and is_upper_bound(denominators, bound))
        denominator_list_upper_bound(bound, List.cons(n, denominators))
    }
}

/// A nonempty lower-bounded denominator list has a lower-bounded head.
theorem denominator_list_lower_bound_head(bound: Nat, n: Nat, denominators: List[Nat]) {
    denominator_list_lower_bound(bound, List.cons(n, denominators)) implies bound <= n
} by {
    if denominator_list_lower_bound(bound, List.cons(n, denominators)) {
        lower_bound_cons_iff[Nat](n, denominators, bound)
        is_lower_bound(List.cons(n, denominators), bound) =
            (bound <= n and is_lower_bound(denominators, bound))
        bound <= n
    }
}

/// The tail of a nonempty lower-bounded denominator list is lower-bounded.
theorem denominator_list_lower_bound_tail(bound: Nat, n: Nat, denominators: List[Nat]) {
    denominator_list_lower_bound(bound, List.cons(n, denominators))
        implies denominator_list_lower_bound(bound, denominators)
} by {
    if denominator_list_lower_bound(bound, List.cons(n, denominators)) {
        lower_bound_cons_iff[Nat](n, denominators, bound)
        is_lower_bound(List.cons(n, denominators), bound) =
            (bound <= n and is_lower_bound(denominators, bound))
        denominator_list_lower_bound(bound, denominators)
    }
}

/// A smaller lower bound is also a lower bound for the same denominators.
theorem denominator_list_lower_bound_monotone(bound: Nat, smaller: Nat,
    denominators: List[Nat]) {
    denominator_list_lower_bound(bound, denominators) and smaller <= bound
        implies denominator_list_lower_bound(smaller, denominators)
} by {
    if denominator_list_lower_bound(bound, denominators) and smaller <= bound {
        lower_bound_monotone[Nat](denominators, bound, smaller)
        is_lower_bound(denominators, smaller)
        denominator_list_lower_bound(smaller, denominators)
    }
}

/// A tail bounded below by `n.suc` is also bounded below by `n`.
theorem denominator_list_lower_bound_suc_imp_lower_bound(n: Nat,
    denominators: List[Nat]) {
    denominator_list_lower_bound(n.suc, denominators)
        implies denominator_list_lower_bound(n, denominators)
} by {
    if denominator_list_lower_bound(n.suc, denominators) {
        lt_suc(n)
        n <= n.suc
        lower_bound_monotone[Nat](denominators, n.suc, n)
        denominator_list_lower_bound(n, denominators)
    }
}

/// Consing `n` onto a tail bounded below by `n.suc` preserves lower bound `n`.
theorem denominator_list_lower_bound_cons_self_of_suc(n: Nat,
    denominators: List[Nat]) {
    denominator_list_lower_bound(n.suc, denominators)
        implies denominator_list_lower_bound(n, List.cons(n, denominators))
} by {
    if denominator_list_lower_bound(n.suc, denominators) {
        denominator_list_lower_bound_suc_imp_lower_bound(n, denominators)
        denominator_list_lower_bound(n, denominators)
        denominator_list_lower_bound_cons(n, n, denominators)
        denominator_list_lower_bound(n, List.cons(n, denominators))
    }
}

/// Inductive predicate for the lower bound of positive denominator lists.
define positive_denominator_list_lower_bound_one_pred(denominators: List[Nat]) -> Bool {
    positive_denominator_list(denominators) implies
        denominator_list_lower_bound(Nat.1, denominators)
}

/// Positive denominator lists have lower bound one.
theorem positive_denominator_list_lower_bound_one_all {
    forall(denominators: List[Nat]) {
        positive_denominator_list_lower_bound_one_pred(denominators)
    }
} by {
    if positive_denominator_list(List.nil[Nat]) {
        denominator_list_lower_bound_nil(Nat.1)
    }
    positive_denominator_list_lower_bound_one_pred(List.nil[Nat]) =
        (positive_denominator_list(List.nil[Nat]) implies
            denominator_list_lower_bound(Nat.1, List.nil[Nat]))
    positive_denominator_list_lower_bound_one_pred(List.nil[Nat])

    forall(head: Nat, tail: List[Nat]) {
        if positive_denominator_list_lower_bound_one_pred(tail) {
            if positive_denominator_list(List.cons(head, tail)) {
                positive_denominator_list_head(head, tail)
                Nat.0 < head
                Nat.1 <= head
                positive_denominator_list_tail(head, tail)
                positive_denominator_list(tail)
                positive_denominator_list_lower_bound_one_pred(tail) =
                    (positive_denominator_list(tail) implies
                        denominator_list_lower_bound(Nat.1, tail))
                denominator_list_lower_bound(Nat.1, tail)
                denominator_list_lower_bound_cons(Nat.1, head, tail)
                denominator_list_lower_bound(Nat.1, List.cons(head, tail))
            }
            positive_denominator_list_lower_bound_one_pred(List.cons(head, tail)) =
                (positive_denominator_list(List.cons(head, tail)) implies
                    denominator_list_lower_bound(Nat.1, List.cons(head, tail)))
            positive_denominator_list_lower_bound_one_pred(List.cons(head, tail))
        }
    }
    forall(head: Nat, tail: List[Nat]) {
        positive_denominator_list_lower_bound_one_pred(tail) implies positive_denominator_list_lower_bound_one_pred(List.cons(head, tail))
    }
    List.induction(function(items: List[Nat]) {
        positive_denominator_list_lower_bound_one_pred(items)
    })
}

/// Positive denominator lists have lower bound one.
theorem positive_denominator_list_lower_bound_one(denominators: List[Nat]) {
    positive_denominator_list(denominators) implies
        denominator_list_lower_bound(Nat.1, denominators)
} by {
    positive_denominator_list_lower_bound_one_all
    positive_denominator_list_lower_bound_one_pred(denominators)
    positive_denominator_list_lower_bound_one_pred(denominators) =
        (positive_denominator_list(denominators) implies
            denominator_list_lower_bound(Nat.1, denominators))
}

/// Every member of a positive denominator list is positive.
theorem positive_denominator_list_contains_positive(denominators: List[Nat], n: Nat) {
    positive_denominator_list(denominators) and denominators.contains(n) implies Nat.0 < n
} by {
    if positive_denominator_list(denominators) and denominators.contains(n) {
        positive_denominator_list_lower_bound_one(denominators)
        denominator_list_lower_bound(Nat.1, denominators)
        lower_bound_contains[Nat](denominators, Nat.1, n)
        Nat.1 <= n
        Nat.0 < n
    }
}

/// A lower bound above `n` makes `n` fresh for the denominator list.
theorem denominator_list_lower_bound_suc_not_contains(n: Nat,
    denominators: List[Nat]) {
    denominator_list_lower_bound(n.suc, denominators)
        implies not denominators.contains(n)
} by {
    if denominator_list_lower_bound(n.suc, denominators) and
        denominators.contains(n) {
        lower_bound_contains[Nat](denominators, n.suc, n)
        n.suc <= n
        lte_imp_not_lt(n.suc, n)
        not n < n.suc
        lt_suc(n)
        n < n.suc
        false
    }
}

/// A nonempty upper-bounded denominator list has an upper-bounded head.
theorem denominator_list_upper_bound_head(bound: Nat, n: Nat, denominators: List[Nat]) {
    denominator_list_upper_bound(bound, List.cons(n, denominators)) implies n <= bound
} by {
    if denominator_list_upper_bound(bound, List.cons(n, denominators)) {
        upper_bound_cons_iff[Nat](n, denominators, bound)
        is_upper_bound(List.cons(n, denominators), bound) =
            (n <= bound and is_upper_bound(denominators, bound))
        n <= bound
    }
}

/// The tail of a nonempty upper-bounded denominator list is upper-bounded.
theorem denominator_list_upper_bound_tail(bound: Nat, n: Nat, denominators: List[Nat]) {
    denominator_list_upper_bound(bound, List.cons(n, denominators))
        implies denominator_list_upper_bound(bound, denominators)
} by {
    if denominator_list_upper_bound(bound, List.cons(n, denominators)) {
        upper_bound_cons_iff[Nat](n, denominators, bound)
        is_upper_bound(List.cons(n, denominators), bound) =
            (n <= bound and is_upper_bound(denominators, bound))
        denominator_list_upper_bound(bound, denominators)
    }
}

/// Lower bounds are preserved by appending denominator lists.
theorem denominator_list_lower_bound_append(bound: Nat, left: List[Nat], right: List[Nat]) {
    denominator_list_lower_bound(bound, left) and denominator_list_lower_bound(bound, right)
        implies denominator_list_lower_bound(bound, left + right)
} by {
    if denominator_list_lower_bound(bound, left) and denominator_list_lower_bound(bound, right) {
        lower_bound_add_iff[Nat](left, right, bound)
        is_lower_bound(left + right, bound) =
            (is_lower_bound(left, bound) and is_lower_bound(right, bound))
        denominator_list_lower_bound(bound, left + right)
    }
}

/// Upper bounds are preserved by appending denominator lists.
theorem denominator_list_upper_bound_append(bound: Nat, left: List[Nat], right: List[Nat]) {
    denominator_list_upper_bound(bound, left) and denominator_list_upper_bound(bound, right)
        implies denominator_list_upper_bound(bound, left + right)
} by {
    if denominator_list_upper_bound(bound, left) and denominator_list_upper_bound(bound, right) {
        upper_bound_add_iff[Nat](left, right, bound)
        is_upper_bound(left + right, bound) =
            (is_upper_bound(left, bound) and is_upper_bound(right, bound))
        denominator_list_upper_bound(bound, left + right)
    }
}

/// Lower bounds are preserved by filtering denominator lists.
theorem denominator_list_lower_bound_filter(bound: Nat, denominators: List[Nat],
    pred: Nat -> Bool) {
    denominator_list_lower_bound(bound, denominators)
        implies denominator_list_lower_bound(bound, denominators.filter(pred))
} by {
    if denominator_list_lower_bound(bound, denominators) {
        forall(n: Nat) {
            if denominators.filter(pred).contains(n) {
                filter_contained_by_and[Nat](denominators, pred, n)
                denominators.contains(n)
                lower_bound_contains[Nat](denominators, bound, n)
                bound <= n
            }
        }
        list_lower_bound[Nat](denominators.filter(pred), bound) =
            forall(n: Nat) {
                denominators.filter(pred).contains(n) implies bound <= n
            }
        list_lower_bound(denominators.filter(pred), bound)
        list_lower_bound_imp_lower_bound[Nat](denominators.filter(pred), bound)
        is_lower_bound(denominators.filter(pred), bound)
        denominator_list_lower_bound(bound, denominators.filter(pred))
    }
}

/// Upper bounds are preserved by filtering denominator lists.
theorem denominator_list_upper_bound_filter(bound: Nat, denominators: List[Nat],
    pred: Nat -> Bool) {
    denominator_list_upper_bound(bound, denominators)
        implies denominator_list_upper_bound(bound, denominators.filter(pred))
} by {
    if denominator_list_upper_bound(bound, denominators) {
        forall(n: Nat) {
            if denominators.filter(pred).contains(n) {
                filter_contained_by_and[Nat](denominators, pred, n)
                denominators.contains(n)
                upper_bound_contains[Nat](denominators, bound, n)
                n <= bound
            }
        }
        list_upper_bound[Nat](denominators.filter(pred), bound) =
            forall(n: Nat) {
                denominators.filter(pred).contains(n) implies n <= bound
            }
        list_upper_bound(denominators.filter(pred), bound)
        list_upper_bound_imp_upper_bound[Nat](denominators.filter(pred), bound)
        is_upper_bound(denominators.filter(pred), bound)
        denominator_list_upper_bound(bound, denominators.filter(pred))
    }
}

/// Inductive predicate for embedding positive denominator sums into the reals.
define unit_fraction_sum_to_real_pred(denominators: List[Nat]) -> Bool {
    positive_denominator_list(denominators) implies
        Real.from_rat(unit_fraction_sum(denominators)) =
            real_unit_fraction_sum(denominators)
}

/// Every positive denominator sum embeds into the matching real sum.
theorem unit_fraction_sum_to_real_all {
    forall(denominators: List[Nat]) {
        unit_fraction_sum_to_real_pred(denominators)
    }
} by {
    if positive_denominator_list(List.nil[Nat]) {
        unit_fraction_sum_nil
        real_unit_fraction_sum_nil
        unit_fraction_sum(List.nil[Nat]) = Rat.0
        real_unit_fraction_sum(List.nil[Nat]) = Real.0
        Real.from_rat(unit_fraction_sum(List.nil[Nat])) = Real.from_rat(Rat.0)
        Real.from_rat(Rat.0) = Real.0
        Real.from_rat(unit_fraction_sum(List.nil[Nat])) =
            real_unit_fraction_sum(List.nil[Nat])
    }
    unit_fraction_sum_to_real_pred(List.nil[Nat]) =
        (positive_denominator_list(List.nil[Nat]) implies
            Real.from_rat(unit_fraction_sum(List.nil[Nat])) =
                real_unit_fraction_sum(List.nil[Nat]))
    unit_fraction_sum_to_real_pred(List.nil[Nat])

    forall(head: Nat, tail: List[Nat]) {
        if unit_fraction_sum_to_real_pred(tail) {
            if positive_denominator_list(List.cons(head, tail)) {
                positive_denominator_list_head(head, tail)
                Nat.0 < head
                positive_denominator_list_tail(head, tail)
                positive_denominator_list(tail)
                unit_fraction_sum_to_real_pred(tail) =
                    (positive_denominator_list(tail) implies
                        Real.from_rat(unit_fraction_sum(tail)) =
                            real_unit_fraction_sum(tail))
                Real.from_rat(unit_fraction_sum(tail)) =
                    real_unit_fraction_sum(tail)
                unit_fraction_to_real(head)
                Real.from_rat(unit_fraction(head)) = real_unit_fraction(head)
                unit_fraction_sum_cons(head, tail)
                real_unit_fraction_sum_cons(head, tail)
                unit_fraction_sum(List.cons(head, tail)) =
                    unit_fraction(head) + unit_fraction_sum(tail)
                real_unit_fraction_sum(List.cons(head, tail)) =
                    real_unit_fraction(head) + real_unit_fraction_sum(tail)
                add_from_rat(unit_fraction(head), unit_fraction_sum(tail))
                Real.from_rat(unit_fraction(head)) +
                    Real.from_rat(unit_fraction_sum(tail)) =
                    Real.from_rat(unit_fraction(head) + unit_fraction_sum(tail))
                Real.from_rat(unit_fraction(head) + unit_fraction_sum(tail)) =
                    real_unit_fraction(head) + real_unit_fraction_sum(tail)
                Real.from_rat(unit_fraction_sum(List.cons(head, tail))) =
                    real_unit_fraction_sum(List.cons(head, tail))
            }
            unit_fraction_sum_to_real_pred(List.cons(head, tail)) =
                (positive_denominator_list(List.cons(head, tail)) implies
                    Real.from_rat(unit_fraction_sum(List.cons(head, tail))) =
                        real_unit_fraction_sum(List.cons(head, tail)))
            unit_fraction_sum_to_real_pred(List.cons(head, tail))
        }
    }
    forall(head: Nat, tail: List[Nat]) {
        unit_fraction_sum_to_real_pred(tail) implies unit_fraction_sum_to_real_pred(List.cons(head, tail))
    }
    List.induction(function(items: List[Nat]) { unit_fraction_sum_to_real_pred(items) })
}

/// A finite sum over positive denominators embeds into the matching real sum.
theorem unit_fraction_sum_to_real(denominators: List[Nat]) {
    positive_denominator_list(denominators) implies
        Real.from_rat(unit_fraction_sum(denominators)) =
            real_unit_fraction_sum(denominators)
} by {
    unit_fraction_sum_to_real_all
    unit_fraction_sum_to_real_pred(denominators)
    unit_fraction_sum_to_real_pred(denominators) =
        (positive_denominator_list(denominators) implies
            Real.from_rat(unit_fraction_sum(denominators)) =
                real_unit_fraction_sum(denominators))
}

/// Inductive predicate for nonnegativity of unit-fraction sums.
define unit_fraction_sum_nonnegative_pred(denominators: List[Nat]) -> Bool {
    positive_denominator_list(denominators) implies Rat.0 <= unit_fraction_sum(denominators)
}

/// The nonnegativity predicate holds for every denominator list.
theorem unit_fraction_sum_nonnegative_all {
    forall(denominators: List[Nat]) {
        unit_fraction_sum_nonnegative_pred(denominators)
    }
} by {
    unit_fraction_sum_nil
    unit_fraction_sum(List.nil[Nat]) = Rat.0
    Rat.0 <= unit_fraction_sum(List.nil[Nat])
    unit_fraction_sum_nonnegative_pred(List.nil[Nat])

    forall(head: Nat, tail: List[Nat]) {
        if unit_fraction_sum_nonnegative_pred(tail) {
            if positive_denominator_list(List.cons(head, tail)) {
                positive_denominator_list_head(head, tail)
                Nat.0 < head
                positive_denominator_list_tail(head, tail)
                positive_denominator_list(tail)
                unit_fraction_sum_nonnegative_pred(tail) =
                    (positive_denominator_list(tail) implies Rat.0 <= unit_fraction_sum(tail))
                Rat.0 <= unit_fraction_sum(tail)
                unit_fraction_positive(head)
                unit_fraction(head).is_positive
                Rat.0 < unit_fraction(head)
                Rat.0 <= unit_fraction(head)
                lte_add_right(Rat.0, unit_fraction(head), unit_fraction_sum(tail))
                Rat.0 + unit_fraction_sum(tail) <= unit_fraction(head) + unit_fraction_sum(tail)
                Rat.0 + unit_fraction_sum(tail) = unit_fraction_sum(tail)
                unit_fraction_sum(tail) <= unit_fraction(head) + unit_fraction_sum(tail)
                lte_trans(Rat.0, unit_fraction_sum(tail),
                    unit_fraction(head) + unit_fraction_sum(tail))
                Rat.0 <= unit_fraction(head) + unit_fraction_sum(tail)
                unit_fraction_sum_cons(head, tail)
                unit_fraction_sum(List.cons(head, tail)) =
                    unit_fraction(head) + unit_fraction_sum(tail)
                Rat.0 <= unit_fraction_sum(List.cons(head, tail))
            }
            unit_fraction_sum_nonnegative_pred(List.cons(head, tail))
        }
    }
    forall(head: Nat, tail: List[Nat]) {
        unit_fraction_sum_nonnegative_pred(tail) implies unit_fraction_sum_nonnegative_pred(List.cons(head, tail))
    }
    List.induction(function(items: List[Nat]) { unit_fraction_sum_nonnegative_pred(items) })
}

/// A unit-fraction sum over positive denominators is nonnegative.
theorem unit_fraction_sum_nonnegative(denominators: List[Nat]) {
    positive_denominator_list(denominators) implies Rat.0 <= unit_fraction_sum(denominators)
} by {
    unit_fraction_sum_nonnegative_all
    unit_fraction_sum_nonnegative_pred(denominators)
    unit_fraction_sum_nonnegative_pred(denominators) =
        (positive_denominator_list(denominators) implies
            Rat.0 <= unit_fraction_sum(denominators))
    unit_fraction_sum_nonnegative_pred(denominators)
}

/// Consing a positive denominator gives a positive unit-fraction sum.
theorem unit_fraction_sum_cons_positive(n: Nat, denominators: List[Nat]) {
    Nat.0 < n and positive_denominator_list(denominators)
        implies unit_fraction_sum(List.cons(n, denominators)).is_positive
} by {
    if Nat.0 < n and positive_denominator_list(denominators) {
        unit_fraction_positive(n)
        unit_fraction(n).is_positive
        unit_fraction_sum_nonnegative(denominators)
        Rat.0 <= unit_fraction_sum(denominators)
        lte_add_left(unit_fraction(n), Rat.0, unit_fraction_sum(denominators))
        unit_fraction(n) + Rat.0 <= unit_fraction(n) + unit_fraction_sum(denominators)
        unit_fraction(n) + Rat.0 = unit_fraction(n)
        unit_fraction(n) <= unit_fraction(n) + unit_fraction_sum(denominators)
        pos_lte(unit_fraction(n), unit_fraction(n) + unit_fraction_sum(denominators))
        (unit_fraction(n) + unit_fraction_sum(denominators)).is_positive
        unit_fraction_sum_cons(n, denominators)
        unit_fraction_sum(List.cons(n, denominators)) =
            unit_fraction(n) + unit_fraction_sum(denominators)
        unit_fraction_sum(List.cons(n, denominators)).is_positive
    }
}

/// A positive singleton denominator list has positive denominators.
theorem positive_denominator_list_singleton(n: Nat) {
    Nat.0 < n implies positive_denominator_list(List.singleton(n))
} by {
    if Nat.0 < n {
        positive_denominator_list_nil
        positive_denominator_list_cons(n, List.nil[Nat])
    }
}

/// The empty list is a distinct positive denominator list.
theorem egyptian_denominator_list_nil {
    egyptian_denominator_list(List.nil[Nat])
} by {
    positive_denominator_list_nil
    List.nil[Nat].is_unique
}

/// An Egyptian denominator list has unique denominators.
theorem egyptian_denominator_list_unique(denominators: List[Nat]) {
    egyptian_denominator_list(denominators) implies denominators.is_unique
} by {
    if egyptian_denominator_list(denominators) {
        egyptian_denominator_list(denominators) =
            (denominators.is_unique and positive_denominator_list(denominators))
        denominators.is_unique
    }
}

/// An Egyptian denominator list has positive denominators.
theorem egyptian_denominator_list_positive(denominators: List[Nat]) {
    egyptian_denominator_list(denominators) implies
        positive_denominator_list(denominators)
} by {
    if egyptian_denominator_list(denominators) {
        egyptian_denominator_list(denominators) =
            (denominators.is_unique and positive_denominator_list(denominators))
        positive_denominator_list(denominators)
    }
}

/// A positive singleton denominator is a distinct positive denominator list.
theorem egyptian_denominator_list_singleton(n: Nat) {
    Nat.0 < n implies egyptian_denominator_list(List.singleton(n))
} by {
    if Nat.0 < n {
        singleton_unique[Nat](n)
        List.singleton(n).is_unique
        positive_denominator_list_singleton(n)
        positive_denominator_list(List.singleton(n))
    }
}

lemma egyptian_denominator_list_unique_part(denominators: List[Nat]) {
    egyptian_denominator_list(denominators) implies denominators.is_unique
} by {
    if egyptian_denominator_list(denominators) {
        egyptian_denominator_list(denominators) =
            (denominators.is_unique and positive_denominator_list(denominators))
        denominators.is_unique
    }
}

lemma egyptian_denominator_list_positive_part(denominators: List[Nat]) {
    egyptian_denominator_list(denominators) implies positive_denominator_list(denominators)
} by {
    if egyptian_denominator_list(denominators) {
        egyptian_denominator_list(denominators) =
            (denominators.is_unique and positive_denominator_list(denominators))
        positive_denominator_list(denominators)
    }
}

/// Consing a fresh positive denominator preserves Egyptian denominator lists.
theorem egyptian_denominator_list_cons(n: Nat, denominators: List[Nat]) {
    Nat.0 < n and egyptian_denominator_list(denominators) and
        not denominators.contains(n)
        implies egyptian_denominator_list(List.cons(n, denominators))
} by {
    if Nat.0 < n and egyptian_denominator_list(denominators) and
        not denominators.contains(n) {
        egyptian_denominator_list(denominators) =
            (denominators.is_unique and positive_denominator_list(denominators))
        denominators.is_unique
        positive_denominator_list(denominators)
        cons_unique_of_tail_unique_not_contains[Nat](n, denominators)
        List.cons(n, denominators).is_unique
        positive_denominator_list_cons(n, denominators)
        positive_denominator_list(List.cons(n, denominators))
        egyptian_denominator_list(List.cons(n, denominators)) =
            (List.cons(n, denominators).is_unique and
                positive_denominator_list(List.cons(n, denominators)))
        egyptian_denominator_list(List.cons(n, denominators))
    }
}

/// Appending disjoint Egyptian denominator lists gives an Egyptian denominator list.
theorem egyptian_denominator_list_append(left: List[Nat], right: List[Nat]) {
    egyptian_denominator_list(left) and egyptian_denominator_list(right) and
        (forall(n: Nat) { not (left.contains(n) and right.contains(n)) })
        implies egyptian_denominator_list(left + right)
} by {
    if egyptian_denominator_list(left) and egyptian_denominator_list(right) and
        (forall(n: Nat) { not (left.contains(n) and right.contains(n)) }) {
        egyptian_denominator_list(left) =
            (left.is_unique and positive_denominator_list(left))
        left.is_unique
        positive_denominator_list(left)
        egyptian_denominator_list(right) =
            (right.is_unique and positive_denominator_list(right))
        right.is_unique
        positive_denominator_list(right)
        unique_list_sum[Nat](left, right)
        (left + right).is_unique
        positive_denominator_list_append(left, right)
        positive_denominator_list(left + right)
        egyptian_denominator_list(left + right) =
            ((left + right).is_unique and positive_denominator_list(left + right))
        egyptian_denominator_list(left + right)
    }
}

/// Zero is represented by the empty sum of unit fractions.
theorem is_egyptian_fraction_zero {
    is_egyptian_fraction(Rat.0)
} by {
    egyptian_denominator_list_nil
    unit_fraction_sum_nil
    Rat.0 = unit_fraction_sum(List.nil[Nat])
    exists(denominators: List[Nat]) {
        egyptian_denominator_list(denominators) and
            Rat.0 = unit_fraction_sum(denominators)
    }
}

/// A lower-bounded Egyptian fraction is an Egyptian fraction.
theorem is_egyptian_fraction_with_lower_bound_imp_egyptian(q: Rat, bound: Nat) {
    is_egyptian_fraction_with_lower_bound(q, bound) implies is_egyptian_fraction(q)
} by {
    if is_egyptian_fraction_with_lower_bound(q, bound) {
        let denominators: List[Nat] satisfy {
            egyptian_denominator_list(denominators) and
                denominator_list_lower_bound(bound, denominators) and
                q = unit_fraction_sum(denominators)
        }
        exists(witness: List[Nat]) {
            egyptian_denominator_list(witness) and
                q = unit_fraction_sum(witness)
        }
    }
}

/// Zero is represented by the empty sum with any denominator lower bound.
theorem is_egyptian_fraction_with_lower_bound_zero(bound: Nat) {
    is_egyptian_fraction_with_lower_bound(Rat.0, bound)
} by {
    egyptian_denominator_list_nil
    denominator_list_lower_bound_nil(bound)
    unit_fraction_sum_nil
    Rat.0 = unit_fraction_sum(List.nil[Nat])
    exists(denominators: List[Nat]) {
        egyptian_denominator_list(denominators) and
            denominator_list_lower_bound(bound, denominators) and
            Rat.0 = unit_fraction_sum(denominators)
    }
}

/// A unit-fraction sum over Egyptian denominators is an Egyptian fraction.
theorem unit_fraction_sum_is_egyptian(denominators: List[Nat]) {
    egyptian_denominator_list(denominators)
        implies is_egyptian_fraction(unit_fraction_sum(denominators))
} by {
    if egyptian_denominator_list(denominators) {
        exists(witness: List[Nat]) {
            egyptian_denominator_list(witness) and
                unit_fraction_sum(denominators) = unit_fraction_sum(witness)
        }
    }
}

/// A unit-fraction sum over lower-bounded Egyptian denominators is a
/// lower-bounded Egyptian fraction.
theorem unit_fraction_sum_is_egyptian_with_lower_bound(denominators: List[Nat],
    bound: Nat) {
    egyptian_denominator_list(denominators) and
        denominator_list_lower_bound(bound, denominators)
        implies is_egyptian_fraction_with_lower_bound(unit_fraction_sum(denominators),
            bound)
} by {
    if egyptian_denominator_list(denominators) and
        denominator_list_lower_bound(bound, denominators) {
        exists(witness: List[Nat]) {
            egyptian_denominator_list(witness) and
                denominator_list_lower_bound(bound, witness) and
                unit_fraction_sum(denominators) = unit_fraction_sum(witness)
        }
    }
}

/// A smaller denominator lower bound preserves a lower-bounded Egyptian
/// representation.
theorem is_egyptian_fraction_with_lower_bound_monotone(q: Rat, bound: Nat,
    smaller: Nat) {
    is_egyptian_fraction_with_lower_bound(q, bound) and smaller <= bound
        implies is_egyptian_fraction_with_lower_bound(q, smaller)
} by {
    if is_egyptian_fraction_with_lower_bound(q, bound) and smaller <= bound {
        let denominators: List[Nat] satisfy {
            egyptian_denominator_list(denominators) and
                denominator_list_lower_bound(bound, denominators) and
                q = unit_fraction_sum(denominators)
        }
        denominator_list_lower_bound_monotone(bound, smaller, denominators)
        denominator_list_lower_bound(smaller, denominators)
        exists(witness: List[Nat]) {
            egyptian_denominator_list(witness) and
                denominator_list_lower_bound(smaller, witness) and
                q = unit_fraction_sum(witness)
        }
    }
}

/// True when two Egyptian representations may be added by appending their
/// disjoint denominator lists.
define egyptian_fraction_add_disjoint_hyp(q: Rat, r: Rat, left: List[Nat],
    right: List[Nat]) -> Bool {
    egyptian_denominator_list(left) and egyptian_denominator_list(right) and
    (forall(n: Nat) { not (left.contains(n) and right.contains(n)) }) and
    q = unit_fraction_sum(left) and r = unit_fraction_sum(right)
}

/// Adding two disjoint Egyptian representations gives an Egyptian fraction.
theorem egyptian_fraction_add_disjoint(q: Rat, r: Rat, left: List[Nat], right: List[Nat]) {
    egyptian_fraction_add_disjoint_hyp(q, r, left, right) implies is_egyptian_fraction(q + r)
} by {
    if egyptian_fraction_add_disjoint_hyp(q, r, left, right) {
        egyptian_fraction_add_disjoint_hyp(q, r, left, right) =
            (egyptian_denominator_list(left) and egyptian_denominator_list(right) and
                (forall(n: Nat) { not (left.contains(n) and right.contains(n)) }) and
                q = unit_fraction_sum(left) and r = unit_fraction_sum(right))
        egyptian_denominator_list(left) and egyptian_denominator_list(right) and
            (forall(n: Nat) { not (left.contains(n) and right.contains(n)) })
        egyptian_denominator_list_append(left, right)
        egyptian_denominator_list(left + right)
        unit_fraction_sum_append(left, right)
        unit_fraction_sum(left + right) = unit_fraction_sum(left) + unit_fraction_sum(right)
        q + r = unit_fraction_sum(left) + unit_fraction_sum(right)
        q + r = unit_fraction_sum(left + right)
        exists(witness: List[Nat]) {
            egyptian_denominator_list(witness) and q + r = unit_fraction_sum(witness)
        }
    }
}

/// Adding two disjoint lower-bounded Egyptian representations gives a
/// lower-bounded Egyptian fraction.
theorem egyptian_fraction_add_disjoint_lower_bound(q: Rat, r: Rat, bound: Nat,
    left: List[Nat], right: List[Nat]) {
    egyptian_denominator_list(left) and egyptian_denominator_list(right) and
        (forall(n: Nat) { not (left.contains(n) and right.contains(n)) }) and
        denominator_list_lower_bound(bound, left) and
        denominator_list_lower_bound(bound, right) and
        q = unit_fraction_sum(left) and r = unit_fraction_sum(right)
        implies is_egyptian_fraction_with_lower_bound(q + r, bound)
} by {
    if egyptian_denominator_list(left) {
        if egyptian_denominator_list(right) {
            if forall(n: Nat) { not (left.contains(n) and right.contains(n)) } {
                if denominator_list_lower_bound(bound, left) {
                    if denominator_list_lower_bound(bound, right) {
                        if q = unit_fraction_sum(left) {
                            if r = unit_fraction_sum(right) {
                                egyptian_denominator_list_append(left, right)
                                egyptian_denominator_list(left + right)
                                denominator_list_lower_bound_append(bound, left, right)
                                denominator_list_lower_bound(bound, left + right)
                                unit_fraction_sum_append(left, right)
                                unit_fraction_sum(left + right) =
                                    unit_fraction_sum(left) + unit_fraction_sum(right)
                                q + r = unit_fraction_sum(left) + r
                                q + r = unit_fraction_sum(left) + unit_fraction_sum(right)
                                q + r = unit_fraction_sum(left + right)
                                exists(witness: List[Nat]) {
                                    egyptian_denominator_list(witness) and
                                        denominator_list_lower_bound(bound, witness) and
                                        q + r = unit_fraction_sum(witness)
                                }
                            }
                        }
                    }
                }
            }
        }
    }
}

/// Adding a fresh positive denominator to an Egyptian representation gives an
/// Egyptian fraction.
theorem egyptian_fraction_add_fresh_unit(q: Rat, n: Nat, denominators: List[Nat]) {
    Nat.0 < n and egyptian_denominator_list(denominators) and
        not denominators.contains(n) and q = unit_fraction_sum(denominators)
        implies is_egyptian_fraction(unit_fraction(n) + q)
} by {
    if Nat.0 < n and egyptian_denominator_list(denominators) and
        not denominators.contains(n) and q = unit_fraction_sum(denominators) {
        egyptian_denominator_list_cons(n, denominators)
        egyptian_denominator_list(List.cons(n, denominators))
        unit_fraction_sum_cons(n, denominators)
        unit_fraction_sum(List.cons(n, denominators)) =
            unit_fraction(n) + unit_fraction_sum(denominators)
        unit_fraction(n) + q = unit_fraction(n) + unit_fraction_sum(denominators)
        unit_fraction(n) + q = unit_fraction_sum(List.cons(n, denominators))
        exists(witness: List[Nat]) {
            egyptian_denominator_list(witness) and
                unit_fraction(n) + q = unit_fraction_sum(witness)
        }
    }
}

/// Adding a denominator below a represented lower-bound tail is fresh.
theorem egyptian_fraction_add_lower_bound_unit(q: Rat, n: Nat,
    denominators: List[Nat]) {
    Nat.0 < n and egyptian_denominator_list(denominators) and
        denominator_list_lower_bound(n.suc, denominators) and
        q = unit_fraction_sum(denominators)
        implies is_egyptian_fraction(unit_fraction(n) + q)
} by {
    if Nat.0 < n and egyptian_denominator_list(denominators) and
        denominator_list_lower_bound(n.suc, denominators) and
        q = unit_fraction_sum(denominators) {
        denominator_list_lower_bound_suc_not_contains(n, denominators)
        not denominators.contains(n)
        egyptian_fraction_add_fresh_unit(q, n, denominators)
        is_egyptian_fraction(unit_fraction(n) + q)
    }
}

/// A fresh Egyptian representation of a bounded-step remainder represents the
/// original rational.
theorem egyptian_fraction_of_bounded_step_remainder(q: Rat, n: Nat,
    denominators: List[Nat]) {
    Nat.0 < n and unit_fraction_bounded_step(q, n) and
        egyptian_denominator_list(denominators) and
        not denominators.contains(n) and
        q - unit_fraction(n) = unit_fraction_sum(denominators)
        implies is_egyptian_fraction(q)
} by {
    if Nat.0 < n and unit_fraction_bounded_step(q, n) and
        egyptian_denominator_list(denominators) and
        not denominators.contains(n) and
        q - unit_fraction(n) = unit_fraction_sum(denominators) {
        egyptian_fraction_add_fresh_unit(q - unit_fraction(n), n, denominators)
        is_egyptian_fraction(unit_fraction(n) + (q - unit_fraction(n)))
        unit_fraction(n) + (q - unit_fraction(n)) = q
        is_egyptian_fraction(q)
    }
}

/// A lower-bounded Egyptian representation of a unit-fraction remainder
/// represents the original rational.
theorem egyptian_fraction_of_lower_bound_remainder(q: Rat, n: Nat,
    denominators: List[Nat]) {
    Nat.0 < n and egyptian_denominator_list(denominators) and
        denominator_list_lower_bound(n.suc, denominators) and
        q - unit_fraction(n) = unit_fraction_sum(denominators)
        implies is_egyptian_fraction(q)
} by {
    if Nat.0 < n {
        if egyptian_denominator_list(denominators) {
            if denominator_list_lower_bound(n.suc, denominators) {
                if q - unit_fraction(n) = unit_fraction_sum(denominators) {
                    denominator_list_lower_bound_suc_not_contains(n, denominators)
                    not denominators.contains(n)
                    let remainder = q - unit_fraction(n)
                    remainder = unit_fraction_sum(denominators)
                    egyptian_denominator_list_unique(denominators)
                    denominators.is_unique
                    egyptian_denominator_list_positive(denominators)
                    positive_denominator_list(denominators)
                    cons_unique_of_tail_unique_not_contains[Nat](n, denominators)
                    List.cons(n, denominators).is_unique
                    positive_denominator_list_cons(n, denominators)
                    positive_denominator_list(List.cons(n, denominators))
                    egyptian_denominator_list(List.cons(n, denominators)) =
                        (List.cons(n, denominators).is_unique and
                            positive_denominator_list(List.cons(n, denominators)))
                    egyptian_denominator_list(List.cons(n, denominators))
                    unit_fraction_sum_cons(n, denominators)
                    unit_fraction_sum(List.cons(n, denominators)) =
                        unit_fraction(n) + unit_fraction_sum(denominators)
                    unit_fraction(n) + remainder =
                        unit_fraction(n) + unit_fraction_sum(denominators)
                    unit_fraction(n) + remainder =
                        unit_fraction_sum(List.cons(n, denominators))
                    exists(witness: List[Nat]) {
                        egyptian_denominator_list(witness) and
                            unit_fraction(n) + remainder = unit_fraction_sum(witness)
                    }
                    is_egyptian_fraction(unit_fraction(n) + remainder)
                    unit_fraction(n) + remainder = q
                    is_egyptian_fraction(q)
                }
            }
        }
    }
}

/// Adding a denominator below a represented lower-bound tail preserves a
/// lower-bounded Egyptian representation at the current denominator.
theorem lower_bounded_egyptian_fraction_add_lower_bound_unit(q: Rat, n: Nat,
    denominators: List[Nat]) {
    Nat.0 < n and egyptian_denominator_list(denominators) and
        denominator_list_lower_bound(n.suc, denominators) and
        q = unit_fraction_sum(denominators)
        implies is_lower_bounded_egyptian_fraction(unit_fraction(n) + q, n)
} by {
    if Nat.0 < n and egyptian_denominator_list(denominators) and
        denominator_list_lower_bound(n.suc, denominators) and
        q = unit_fraction_sum(denominators) {
        denominator_list_lower_bound_suc_not_contains(n, denominators)
        not denominators.contains(n)
        egyptian_denominator_list_cons(n, denominators)
        egyptian_denominator_list(List.cons(n, denominators))
        denominator_list_lower_bound_cons_self_of_suc(n, denominators)
        denominator_list_lower_bound(n, List.cons(n, denominators))
        unit_fraction_sum_cons(n, denominators)
        unit_fraction_sum(List.cons(n, denominators)) =
            unit_fraction(n) + unit_fraction_sum(denominators)
        unit_fraction_sum(List.cons(n, denominators)) = unit_fraction(n) + q
        unit_fraction(n) + q = unit_fraction_sum(List.cons(n, denominators))
        exists(witness: List[Nat]) {
            egyptian_denominator_list(witness) and
            denominator_list_lower_bound(n, witness) and
            unit_fraction(n) + q = unit_fraction_sum(witness)
        }
        is_lower_bounded_egyptian_fraction(unit_fraction(n) + q, n)
    }
}

/// A lower-bounded Egyptian representation is in particular Egyptian.
theorem lower_bounded_egyptian_fraction_is_egyptian(q: Rat, bound: Nat) {
    is_lower_bounded_egyptian_fraction(q, bound) implies is_egyptian_fraction(q)
} by {
    if is_lower_bounded_egyptian_fraction(q, bound) {
        is_lower_bounded_egyptian_fraction(q, bound) =
            exists(denominators: List[Nat]) {
                egyptian_denominator_list(denominators) and
                denominator_list_lower_bound(bound, denominators) and
                q = unit_fraction_sum(denominators)
            }
        let denominators: List[Nat] satisfy {
            egyptian_denominator_list(denominators) and
            denominator_list_lower_bound(bound, denominators) and
            q = unit_fraction_sum(denominators)
        }
        exists(witness: List[Nat]) {
            egyptian_denominator_list(witness) and q = unit_fraction_sum(witness)
        }
        is_egyptian_fraction(q)
    }
}

/// Every positive unit fraction is an Egyptian fraction.
theorem unit_fraction_is_egyptian(n: Nat) {
    Nat.0 < n implies is_egyptian_fraction(unit_fraction(n))
} by {
    if Nat.0 < n {
        egyptian_denominator_list_singleton(n)
        unit_fraction_sum_singleton(n)
        unit_fraction(n) = unit_fraction_sum(List.singleton(n))
        exists(denominators: List[Nat]) {
            egyptian_denominator_list(denominators) and
                unit_fraction(n) = unit_fraction_sum(denominators)
        }
    }
}

/// A positive unit fraction has a lower-bounded Egyptian representation
/// whenever its denominator is above the bound.
theorem unit_fraction_is_egyptian_with_lower_bound(n: Nat, bound: Nat) {
    Nat.0 < n and bound <= n
        implies is_egyptian_fraction_with_lower_bound(unit_fraction(n), bound)
} by {
    if Nat.0 < n and bound <= n {
        egyptian_denominator_list_singleton(n)
        lower_bound_singleton_iff[Nat](n, bound)
        is_lower_bound(List.singleton(n), bound) = (bound <= n)
        denominator_list_lower_bound(bound, List.singleton(n))
        unit_fraction_sum_singleton(n)
        unit_fraction(n) = unit_fraction_sum(List.singleton(n))
        exists(denominators: List[Nat]) {
            egyptian_denominator_list(denominators) and
                denominator_list_lower_bound(bound, denominators) and
                unit_fraction(n) = unit_fraction_sum(denominators)
        }
    }
}

/// A lower-bounded Egyptian representation of a remainder extends to a
/// lower-bounded representation of the original rational.
theorem egyptian_fraction_with_lower_bound_of_lower_bound_remainder(q: Rat,
    n: Nat, denominators: List[Nat]) {
    Nat.0 < n and egyptian_denominator_list(denominators) and
        denominator_list_lower_bound(n.suc, denominators) and
        q - unit_fraction(n) = unit_fraction_sum(denominators)
        implies is_egyptian_fraction_with_lower_bound(q, n)
} by {
    if Nat.0 < n {
        if egyptian_denominator_list(denominators) {
            if denominator_list_lower_bound(n.suc, denominators) {
                if q - unit_fraction(n) = unit_fraction_sum(denominators) {
                    denominator_list_lower_bound_suc_not_contains(n, denominators)
                    not denominators.contains(n)
                    egyptian_denominator_list_cons(n, denominators)
                    egyptian_denominator_list(List.cons(n, denominators))
                    lt_suc(n)
                    n <= n.suc
                    denominator_list_lower_bound_monotone(n.suc, n, denominators)
                    denominator_list_lower_bound(n, denominators)
                    n <= n
                    denominator_list_lower_bound_cons(n, n, denominators)
                    denominator_list_lower_bound(n, List.cons(n, denominators))
                    let remainder = q - unit_fraction(n)
                    remainder = unit_fraction_sum(denominators)
                    unit_fraction_sum_cons(n, denominators)
                    unit_fraction_sum(List.cons(n, denominators)) =
                        unit_fraction(n) + unit_fraction_sum(denominators)
                    unit_fraction(n) + remainder =
                        unit_fraction(n) + unit_fraction_sum(denominators)
                    unit_fraction(n) + remainder =
                        unit_fraction_sum(List.cons(n, denominators))
                    unit_fraction(n) + remainder = q
                    q = unit_fraction_sum(List.cons(n, denominators))
                    exists(witness: List[Nat]) {
                        egyptian_denominator_list(witness) and
                            denominator_list_lower_bound(n, witness) and
                            q = unit_fraction_sum(witness)
                    }
                }
            }
        }
    }
}

/// A lower-bounded Egyptian representation of a remainder extends to a
/// lower-bounded representation of the original rational.
theorem egyptian_fraction_with_lower_bound_of_remainder(q: Rat, n: Nat) {
    Nat.0 < n and
        is_egyptian_fraction_with_lower_bound(q - unit_fraction(n), n.suc)
        implies is_egyptian_fraction_with_lower_bound(q, n)
} by {
    if Nat.0 < n and
        is_egyptian_fraction_with_lower_bound(q - unit_fraction(n), n.suc) {
        let denominators: List[Nat] satisfy {
            egyptian_denominator_list(denominators) and
                denominator_list_lower_bound(n.suc, denominators) and
                q - unit_fraction(n) = unit_fraction_sum(denominators)
        }
        egyptian_fraction_with_lower_bound_of_lower_bound_remainder(q, n, denominators)
        is_egyptian_fraction_with_lower_bound(q, n)
    }
}

/// One lower-bound predecessor case of bounded Egyptian-fraction existence at a
/// fixed numerator measure.
define positive_lt_unit_fraction_by_num_prev_case(k: Nat, q: Rat,
    prev: Nat) -> Bool {
    q.is_positive and q < unit_fraction(prev) and
        q.num = Int.from_nat(k.suc)
        implies is_egyptian_fraction_with_lower_bound(q, prev.suc)
}

/// One rational case of bounded Egyptian-fraction existence at a fixed
/// numerator measure.
define positive_lt_unit_fraction_by_num_case(k: Nat, q: Rat) -> Bool {
    forall(prev: Nat) {
        positive_lt_unit_fraction_by_num_prev_case(k, q, prev)
    }
}

/// Introduction rule for one lower-bound predecessor case.
theorem positive_lt_unit_fraction_by_num_prev_case_intro(k: Nat, q: Rat,
    prev: Nat) {
    (q.is_positive and q < unit_fraction(prev) and
        q.num = Int.from_nat(k.suc)
        implies is_egyptian_fraction_with_lower_bound(q, prev.suc))
        implies positive_lt_unit_fraction_by_num_prev_case(k, q, prev)
} by {
    if q.is_positive and q < unit_fraction(prev) and
        q.num = Int.from_nat(k.suc)
        implies is_egyptian_fraction_with_lower_bound(q, prev.suc) {
        positive_lt_unit_fraction_by_num_prev_case(k, q, prev) =
            (q.is_positive and q < unit_fraction(prev) and
                q.num = Int.from_nat(k.suc)
                implies is_egyptian_fraction_with_lower_bound(q, prev.suc))
        positive_lt_unit_fraction_by_num_prev_case(k, q, prev)
    }
}

/// Elimination rule for one lower-bound predecessor case.
theorem positive_lt_unit_fraction_by_num_prev_case_apply(k: Nat, q: Rat,
    prev: Nat) {
    positive_lt_unit_fraction_by_num_prev_case(k, q, prev) and
        q.is_positive and q < unit_fraction(prev) and
        q.num = Int.from_nat(k.suc)
        implies is_egyptian_fraction_with_lower_bound(q, prev.suc)
} by {
    if positive_lt_unit_fraction_by_num_prev_case(k, q, prev) and
        q.is_positive and q < unit_fraction(prev) and
        q.num = Int.from_nat(k.suc) {
        positive_lt_unit_fraction_by_num_prev_case(k, q, prev) =
            (q.is_positive and q < unit_fraction(prev) and
                q.num = Int.from_nat(k.suc)
                implies is_egyptian_fraction_with_lower_bound(q, prev.suc))
        is_egyptian_fraction_with_lower_bound(q, prev.suc)
    }
}

/// Introduction rule for a fixed rational case.
theorem positive_lt_unit_fraction_by_num_case_intro(k: Nat, q: Rat) {
    forall(prev: Nat) {
        positive_lt_unit_fraction_by_num_prev_case(k, q, prev)
    }
    implies positive_lt_unit_fraction_by_num_case(k, q)
} by {
    if forall(prev: Nat) {
        positive_lt_unit_fraction_by_num_prev_case(k, q, prev)
    } {
        positive_lt_unit_fraction_by_num_case(k, q) =
            forall(lower_prev: Nat) {
                positive_lt_unit_fraction_by_num_prev_case(k, q, lower_prev)
            }
        positive_lt_unit_fraction_by_num_case(k, q)
    }
}

/// A fixed rational case gives the corresponding lower-bounded representation.
theorem positive_lt_unit_fraction_by_num_case_apply(k: Nat, q: Rat, prev: Nat) {
    positive_lt_unit_fraction_by_num_case(k, q) and
        q.is_positive and q < unit_fraction(prev) and
        q.num = Int.from_nat(k.suc)
        implies is_egyptian_fraction_with_lower_bound(q, prev.suc)
} by {
    if positive_lt_unit_fraction_by_num_case(k, q) and
        q.is_positive and q < unit_fraction(prev) and
        q.num = Int.from_nat(k.suc) {
        positive_lt_unit_fraction_by_num_case(k, q) =
            forall(lower_prev: Nat) {
                positive_lt_unit_fraction_by_num_prev_case(k, q, lower_prev)
            }
        positive_lt_unit_fraction_by_num_prev_case(k, q, prev)
        positive_lt_unit_fraction_by_num_prev_case_apply(k, q, prev)
        is_egyptian_fraction_with_lower_bound(q, prev.suc)
    }
}

/// The bounded Egyptian-fraction existence predicate for all rationals at a
/// fixed numerator measure.
define positive_lt_unit_fraction_by_num_all_case(k: Nat) -> Bool {
    forall(q: Rat) {
        positive_lt_unit_fraction_by_num_case(k, q)
    }
}

/// Introduction rule for the all-rational numerator-measure case.
theorem positive_lt_unit_fraction_by_num_all_case_intro(k: Nat) {
    forall(q: Rat) {
        positive_lt_unit_fraction_by_num_case(k, q)
    }
    implies positive_lt_unit_fraction_by_num_all_case(k)
} by {
    if forall(q: Rat) {
        positive_lt_unit_fraction_by_num_case(k, q)
    } {
        positive_lt_unit_fraction_by_num_all_case(k) =
            forall(q: Rat) {
                positive_lt_unit_fraction_by_num_case(k, q)
            }
        positive_lt_unit_fraction_by_num_all_case(k)
    }
}

/// The all-rational numerator-measure case contains each rational case.
theorem positive_lt_unit_fraction_by_num_all_case_apply(k: Nat, q: Rat) {
    positive_lt_unit_fraction_by_num_all_case(k)
        implies positive_lt_unit_fraction_by_num_case(k, q)
} by {
    if positive_lt_unit_fraction_by_num_all_case(k) {
        positive_lt_unit_fraction_by_num_all_case(k) =
            forall(r: Rat) {
                positive_lt_unit_fraction_by_num_case(k, r)
            }
        positive_lt_unit_fraction_by_num_case(k, q)
    }
}

/// Induction step for bounded Egyptian-fraction existence at a numerator
/// measure.
theorem positive_lt_unit_fraction_by_num_all_case_step(m: Nat) {
    true_below(positive_lt_unit_fraction_by_num_all_case, m)
        implies positive_lt_unit_fraction_by_num_all_case(m)
} by {
    if true_below(positive_lt_unit_fraction_by_num_all_case, m) {
            forall(r: Rat) {
                forall(lower_prev: Nat) {
                    if r.is_positive and r < unit_fraction(lower_prev) and
                        r.num = Int.from_nat(m.suc) {
                        if lower_prev = Nat.0 {
                            unit_fraction_zero
                            unit_fraction(lower_prev) = Rat.0
                            pos_imp_zero_lt(r)
                            Rat.0 < r
                            lt_trans(Rat.0, r, Rat.0)
                            Rat.0 < Rat.0
                            not_lt_self(Rat.0)
                            false
                        }
                        lower_prev != Nat.0
                        zero_or_suc(lower_prev)
                        let lower_pred: Nat satisfy {
                            lower_prev = lower_pred.suc
                        }
                        Nat.0 < lower_prev
                        Nat.1 <= lower_prev
                        unit_fraction_lte_of_lte(Nat.1, lower_prev)
                        unit_fraction(lower_prev) <= unit_fraction(Nat.1)
                        unit_fraction_one
                        unit_fraction(lower_prev) <= Rat.1
                        lt_lte_trans(r, unit_fraction(lower_prev), Rat.1)
                        r < Rat.1

                        unit_fraction_ceiling_step_exists(r)
                        let n: Nat satisfy {
                            unit_fraction_ceiling_step(r, n)
                        }
                        unit_fraction_ceiling_step_denominator_positive(r, n)
                        Nat.0 < n
                        unit_fraction_ceiling_step_lte(r, n)
                        unit_fraction(n) <= r
                        if n <= lower_prev {
                            unit_fraction_lte_of_lte(n, lower_prev)
                            unit_fraction(lower_prev) <= unit_fraction(n)
                            lte_trans(unit_fraction(lower_prev), unit_fraction(n),
                                r)
                            unit_fraction(lower_prev) <= r
                            lte_lt_trans(unit_fraction(lower_prev), r,
                                unit_fraction(lower_prev))
                            unit_fraction(lower_prev) < unit_fraction(lower_prev)
                            not_lt_self(unit_fraction(lower_prev))
                            false
                        }
                        not n <= lower_prev
                        lt_or_lte(lower_prev, n)
                        lower_prev < n
                        lt_imp_lte_suc(lower_prev, n)
                        lower_prev.suc <= n

                        if r = unit_fraction(n) {
                            unit_fraction_is_egyptian_with_lower_bound(n,
                                lower_prev.suc)
                            is_egyptian_fraction_with_lower_bound(unit_fraction(n),
                                lower_prev.suc)
                            is_egyptian_fraction_with_lower_bound(r,
                                lower_prev.suc)
                        } else {
                            r != unit_fraction(n)
                            let remainder = r - unit_fraction(n)
                            unit_fraction_ceiling_step_remainder_positive(r, n)
                            (r - unit_fraction(n)).is_positive
                            remainder.is_positive
                            unit_fraction_ceiling_step_remainder_lt_unit(r, n)
                            r - unit_fraction(n) < unit_fraction(n)
                            remainder < unit_fraction(n)
                            remainder.num.is_positive = remainder.is_positive
                            remainder.num.is_positive
                            positive_int_eq_from_nat_suc(remainder.num)
                            let j: Nat satisfy {
                                remainder.num = Int.from_nat(j.suc)
                            }
                            unit_fraction_ceiling_step_remainder_num_lt_num(r, n)
                            (r - unit_fraction(n)).num < r.num
                            remainder.num < r.num
                            Int.from_nat(j.suc) < Int.from_nat(m.suc)
                            Int.from_nat(j.suc) <= Int.from_nat(m.suc)
                            lte_nonnegative_ints_implies_nats(j.suc, m.suc)
                            j.suc <= m.suc
                            lte_cancel_suc(j, m)
                            j <= m
                            if m <= j {
                                lte_antisymm(j, m)
                                j = m
                                Int.from_nat(j.suc) = Int.from_nat(m.suc)
                                Int.from_nat(j.suc) < Int.from_nat(j.suc)
                                lt_not_ref(Int.from_nat(j.suc))
                                false
                            }
                            not m <= j
                            lt_or_lte(j, m)
                            j < m
                            true_below_apply(
                                positive_lt_unit_fraction_by_num_all_case, m, j)
                            positive_lt_unit_fraction_by_num_all_case(j)
                            positive_lt_unit_fraction_by_num_all_case_apply(j,
                                remainder)
                            positive_lt_unit_fraction_by_num_case(j, remainder)
                            positive_lt_unit_fraction_by_num_case_apply(j,
                                remainder, n)
                            is_egyptian_fraction_with_lower_bound(remainder,
                                n.suc)
                            is_egyptian_fraction_with_lower_bound(
                                r - unit_fraction(n), n.suc)
                            egyptian_fraction_with_lower_bound_of_remainder(r, n)
                            is_egyptian_fraction_with_lower_bound(r, n)
                            is_egyptian_fraction_with_lower_bound_monotone(r, n,
                                lower_prev.suc)
                            is_egyptian_fraction_with_lower_bound(r,
                                lower_prev.suc)
                        }
                    }
                    (r.is_positive and r < unit_fraction(lower_prev) and
                        r.num = Int.from_nat(m.suc)
                        implies is_egyptian_fraction_with_lower_bound(r,
                            lower_prev.suc))
                    positive_lt_unit_fraction_by_num_prev_case_intro(m, r,
                        lower_prev)
                    positive_lt_unit_fraction_by_num_prev_case(m, r, lower_prev)
                }
                forall(lower_prev: Nat) {
                    positive_lt_unit_fraction_by_num_prev_case(m, r,
                        lower_prev)
                }
                positive_lt_unit_fraction_by_num_case_intro(m, r)
                    positive_lt_unit_fraction_by_num_case(m, r)
            }
            forall(r: Rat) {
                positive_lt_unit_fraction_by_num_case(m, r)
            }
            positive_lt_unit_fraction_by_num_all_case_intro(m)
            positive_lt_unit_fraction_by_num_all_case(m)
        }
}

/// Bounded Egyptian-fraction existence holds at every numerator measure.
theorem positive_lt_unit_fraction_by_num_all_case_step_all {
    forall(k: Nat) {
        true_below(positive_lt_unit_fraction_by_num_all_case, k)
            implies positive_lt_unit_fraction_by_num_all_case(k)
    }
} by {
    forall(k: Nat) {
        positive_lt_unit_fraction_by_num_all_case_step(k)
    }
}

/// Bounded Egyptian-fraction existence holds at every numerator measure.
theorem positive_lt_unit_fraction_by_num_all_case_all {
    forall(k: Nat) {
        positive_lt_unit_fraction_by_num_all_case(k)
    }
} by {
    let f: Nat -> Bool = positive_lt_unit_fraction_by_num_all_case
    let g: Nat -> Bool = function(k: Nat) {
        true_below(f, k)
    }
    true_below_zero(f)
    true_below(f, Nat.0)
    g(Nat.0)
    forall(k: Nat) {
        if g(k) {
            true_below(f, k)
            true_below(positive_lt_unit_fraction_by_num_all_case, k)
            positive_lt_unit_fraction_by_num_all_case_step(k)
            positive_lt_unit_fraction_by_num_all_case(k)
            f(k)
            true_below_suc_intro(f, k)
            true_below(f, k.suc)
            g(k.suc)
        }
        g(k) implies g(k.suc)
    }
    forall(k: Nat) {
        g(k) implies g(k.suc)
    }
    g(Nat.0) and forall(k: Nat) {
        g(k) implies g(k.suc)
    }
    alt_induction(g)
    forall(k: Nat) {
        g(k)
    }
    forall(k: Nat) {
        true_below(f, k)
        true_below(positive_lt_unit_fraction_by_num_all_case, k)
        positive_lt_unit_fraction_by_num_all_case_step(k)
        positive_lt_unit_fraction_by_num_all_case(k)
    }
}

/// A positive rational below a unit fraction has a lower-bounded Egyptian
/// representation, with the numerator measure supplied explicitly.
theorem positive_lt_unit_fraction_is_egyptian_with_lower_bound_of_num(q: Rat,
    prev: Nat, k: Nat) {
    q.is_positive and q < unit_fraction(prev) and q.num = Int.from_nat(k.suc)
        implies is_egyptian_fraction_with_lower_bound(q, prev.suc)
} by {
    if q.is_positive and q < unit_fraction(prev) and
        q.num = Int.from_nat(k.suc) {
        positive_lt_unit_fraction_by_num_all_case_all
        positive_lt_unit_fraction_by_num_all_case(k)
        positive_lt_unit_fraction_by_num_all_case_apply(k, q)
        positive_lt_unit_fraction_by_num_case(k, q)
        positive_lt_unit_fraction_by_num_case_apply(k, q, prev)
        is_egyptian_fraction_with_lower_bound(q, prev.suc)
    }
}

/// A positive rational below a unit fraction has an Egyptian representation
/// whose denominators are all above the predecessor denominator.
theorem positive_lt_unit_fraction_is_egyptian_with_lower_bound(q: Rat, prev: Nat) {
    q.is_positive and q < unit_fraction(prev)
        implies is_egyptian_fraction_with_lower_bound(q, prev.suc)
} by {
    if q.is_positive and q < unit_fraction(prev) {
        q.num.is_positive = q.is_positive
        q.num.is_positive
        positive_int_eq_from_nat_suc(q.num)
        let k: Nat satisfy {
            q.num = Int.from_nat(k.suc)
        }
        positive_lt_unit_fraction_is_egyptian_with_lower_bound_of_num(q, prev,
            k)
        is_egyptian_fraction_with_lower_bound(q, prev.suc)
    }
}

/// A positive rational below one has an Egyptian representation whose
/// denominators are at least two.
theorem positive_lt_one_is_egyptian_with_lower_bound_two(q: Rat) {
    q.is_positive and q < Rat.1
        implies is_egyptian_fraction_with_lower_bound(q, Nat.2)
} by {
    if q.is_positive and q < Rat.1 {
        unit_fraction_one
        q < unit_fraction(Nat.1)
        positive_lt_unit_fraction_is_egyptian_with_lower_bound(q, Nat.1)
        is_egyptian_fraction_with_lower_bound(q, Nat.1.suc)
        Nat.1.suc = Nat.2
        is_egyptian_fraction_with_lower_bound(q, Nat.2)
    }
}

/// A positive rational below one is an Egyptian fraction.
theorem positive_lt_one_is_egyptian(q: Rat) {
    q.is_positive and q < Rat.1 implies is_egyptian_fraction(q)
} by {
    if q.is_positive and q < Rat.1 {
        positive_lt_one_is_egyptian_with_lower_bound_two(q)
        is_egyptian_fraction_with_lower_bound(q, Nat.2)
        is_egyptian_fraction_with_lower_bound_imp_egyptian(q, Nat.2)
        is_egyptian_fraction(q)
    }
}
