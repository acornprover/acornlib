from nat import Nat, mul_cancel_left, divides_self, divides_lte, divides_symm,
    divides_trans, gcd_mult_left, gcd_divides_left, gcd_divides_right,
    gcd_zero_left, gcd_nonzero_left, add_cancels_left, add_imp_sub,
    lte_ref, lte_trans, lte_add_right, lte_mul_both, mul_to_zero,
    pos_of_ne_zero, zero_or_suc, lt_imp_lte_suc, lt_and_lte, lte_antisymm,
    divides_gcd, add_zero_right, add_zero_left, mul_one_right,
    lte_and_lt, lt_not_ref, add_comm, add_assoc, lt_or_lte, lt_suc_right, mul_comm
from int import Int, from_nat, add_from_nat, mul_nat_from_nat_left,
    mul_nat_from_nat_right, mul_one_left
from list import List, map, sum, is_permutation, permutation_preserves_mapped_sum,
    map_map, map_contains, map_contains_of_contains, sum_map_of_pointwise,
    unique_same_contains_imp_permutation, map_length, map_sum_add, map_add
from data.basic.functions import compose
from algebra.add_semigroup import add_fn
from number_theory.divisor_sum import divisor_list, divisor_list_contains_implies,
    divisor_list_contains_of, divisor_list_is_unique, divisors_up_to,
    divisors_up_to_suc_yes, divisors_up_to_suc_no, divisors_up_to_zero,
    divisors_up_to_complete, divisors_up_to_unique, divisors_up_to_member,
    divisor_sum_fn, divisor_sum_fn_apply, divisor_sum_fn_at_zero
from number_theory.dirichlet import divisor_quotient, divisor_quotient_cofactor,
    divisor_quotient_divides, divisor_quotient_positive
from number_theory.totient import count_coprime_to, count_coprime_to_suc_yes,
    count_coprime_to_suc_no, nat_totient, count_multiples_block_no_div,
    unique_count_one, totient_p_pow, totient_sub_one_pred,
    not_contains_count_zero
from number_theory.mobius_inversion import nat_le_add_right
from data.finite.finite_fiber_partition import locally_injective_map_is_unique
numerals Nat
numerals Int

/// The number of natural numbers `x` in `[0, k)` with `gcd(x, n) = d`. The
/// fibers of the map `x -> gcd(x, n)` on `[0, k)`, used to partition the range
/// by divisor of `n`.
define count_gcd_eq(n: Nat, d: Nat, k: Nat) -> Nat {
    match k {
        Nat.zero {
            Nat.0
        }
        Nat.suc(j) {
            if j.gcd(n) = d {
                count_gcd_eq(n, d, j) + Nat.1
            } else {
                count_gcd_eq(n, d, j)
            }
        }
    }
}

/// Recurrence: stepping `k` past an index with `gcd = d` adds one to the count.
theorem count_gcd_eq_suc_yes(n: Nat, d: Nat, k: Nat) {
    k.gcd(n) = d implies count_gcd_eq(n, d, k.suc) = count_gcd_eq(n, d, k) + Nat.1
}

/// Recurrence: stepping `k` past an index without `gcd = d` leaves the count alone.
theorem count_gcd_eq_suc_no(n: Nat, d: Nat, k: Nat) {
    not (k.gcd(n) = d) implies count_gcd_eq(n, d, k.suc) = count_gcd_eq(n, d, k)
}

/// The number of natural numbers `x` in `[1, k]` with `gcd(x, n) = d`. The
/// same fibers as `count_gcd_eq` but on the closed range `[1, k]`, which makes
/// the divisor-sum partition `sum_{d | n} count_gcd_eq_le(n, d, n) = n`
/// straightforward to prove by induction on the bound.
define count_gcd_eq_le(n: Nat, d: Nat, k: Nat) -> Nat {
    match k {
        Nat.zero {
            Nat.0
        }
        Nat.suc(j) {
            if k.gcd(n) = d {
                count_gcd_eq_le(n, d, j) + Nat.1
            } else {
                count_gcd_eq_le(n, d, j)
            }
        }
    }
}

/// Recurrence: stepping past an index with `gcd = d` adds one to the closed-range count.
theorem count_gcd_eq_le_suc_yes(n: Nat, d: Nat, k: Nat) {
    k.suc.gcd(n) = d implies count_gcd_eq_le(n, d, k.suc) = count_gcd_eq_le(n, d, k) + Nat.1
}

/// Recurrence: stepping past an index without `gcd = d` leaves the closed-range count alone.
theorem count_gcd_eq_le_suc_no(n: Nat, d: Nat, k: Nat) {
    not (k.suc.gcd(n) = d) implies count_gcd_eq_le(n, d, k.suc) = count_gcd_eq_le(n, d, k)
}

/// The gcd of a natural with itself is itself.
theorem gcd_self_local(a: Nat) {
    a.gcd(a) = a
} by {
    gcd_divides_left(a, a)
    a.gcd(a).divides(a)
    divides_self(a)
    a.divides(a)
    divides_gcd(a, a, a)
    a.divides(a.gcd(a))
    divides_symm(a.gcd(a), a)
    a.gcd(a) = a
}

/// Euler's totient at a prime power: `totient(p^k) = p^k - p^(k-1)` for prime `p`.
/// Direct counting: the numbers not coprime to `p^k` are exactly the multiples
/// of `p`, of which there are `p^(k-1)` below `p^k`.
theorem totient_prime_power(p: Nat, k: Nat) {
    Nat.1 <= k and p.is_prime implies (p.pow(k)).totient = p.pow(k) - p.pow(k - Nat.1)
} by {
    if Nat.1 <= k and p.is_prime {
        zero_or_suc(k)
        if k = Nat.0 {
            not (Nat.1 <= Nat.0)
            Nat.1 <= Nat.0
            false
        }
        let q: Nat satisfy { q.suc = k }
        q.suc = k
        totient_p_pow(p, q)
        (p.pow(q.suc)).totient = p.pow(q.suc) - p.pow(q)
        q + Nat.1 = k
        add_imp_sub(q, Nat.1, k)
        k - Nat.1 = q
        p.pow(q.suc) = p.pow(k)
        p.pow(q) = p.pow(k - Nat.1)
        (p.pow(k)).totient = p.pow(k) - p.pow(k - Nat.1)
    }
}

/// The indicator of equality: `1` if `a = b`, `0` otherwise.
define nat_indicator_eq(a: Nat, b: Nat) -> Nat {
    if a = b { Nat.1 } else { Nat.0 }
}

/// The indicator of equality is one at equal arguments.
theorem nat_indicator_eq_yes(a: Nat) {
    nat_indicator_eq(a, a) = Nat.1
}

/// The indicator of equality is symmetric.
theorem nat_indicator_eq_symm(a: Nat, b: Nat) {
    nat_indicator_eq(a, b) = nat_indicator_eq(b, a)
}

/// If `gcd(x, n) = d`, then `d` divides `x`.
theorem gcd_eq_imp_divides(x: Nat, n: Nat, d: Nat) {
    x.gcd(n) = d implies d.divides(x)
} by {
    if x.gcd(n) = d {
        gcd_divides_left(x, n)
        x.gcd(n).divides(x)
        d.divides(x)
    }
}

/// Scaling gcd: for `d | n`, `gcd(d * y, n) = d` iff `y` is coprime to `n / d`.
theorem gcd_mul_coprime_iff(n: Nat, d: Nat, y: Nat) {
    d != Nat.0 and d.divides(n)
        implies (y.coprime(divisor_quotient(n, d)) = ((d * y).gcd(n) = d))
} by {
    if d != Nat.0 and d.divides(n) {
        divisor_quotient_cofactor(n, d)
        d * divisor_quotient(n, d) = n
        gcd_mult_left(y, divisor_quotient(n, d), d)
        d * y.gcd(divisor_quotient(n, d)) = (d * y).gcd(d * divisor_quotient(n, d))
        (d * y).gcd(d * divisor_quotient(n, d)) = (d * y).gcd(n)
        if y.coprime(divisor_quotient(n, d)) {
            y.gcd(divisor_quotient(n, d)) = Nat.1
            d * y.gcd(divisor_quotient(n, d)) = d * Nat.1
            d * Nat.1 = d
            (d * y).gcd(n) = d
        }
        if (d * y).gcd(n) = d {
            (d * y).gcd(d * divisor_quotient(n, d)) = d
            d * y.gcd(divisor_quotient(n, d)) = d
            d * Nat.1 = d
            mul_cancel_left(d, y.gcd(divisor_quotient(n, d)), Nat.1)
            y.gcd(divisor_quotient(n, d)) = Nat.1
            y.coprime(divisor_quotient(n, d))
        }
        y.coprime(divisor_quotient(n, d)) = ((d * y).gcd(n) = d)
    }
}

/// For `0 < j < d`, the index `d * k + j` is not a multiple of `d`, so its
/// gcd with `n` cannot equal `d`.
theorem gcd_eq_block_mid_no(n: Nat, d: Nat, k: Nat, j: Nat) {
    d != Nat.0 and Nat.0 < j and j < d
        implies not ((d * k + j).gcd(n) = d)
} by {
    if d != Nat.0 and Nat.0 < j and j < d {
        count_multiples_block_no_div(d, d * k, j)
        not d.divides(d * k + j)
        if (d * k + j).gcd(n) = d {
            gcd_eq_imp_divides(d * k + j, n, d)
            d.divides(d * k + j)
            false
        }
        not ((d * k + j).gcd(n) = d)
    }
}

/// Inductive predicate for the aligned-block walk: with `k` coprime to
/// `n / d`, each of the `d` indices after `d * k` adds exactly one new element
/// with gcd equal to `d` (the index `d * k` itself).
define gcd_eq_block_pred(n: Nat, d: Nat, k: Nat, q: Nat) -> (Nat -> Bool) {
    function(j: Nat) {
        k.coprime(q) and j <= d and Nat.0 < j
            implies count_gcd_eq(n, d, d * k + j) = count_gcd_eq(n, d, d * k) + Nat.1
    }
}

/// Inductive step for the aligned-block walk, coprime case.
theorem gcd_eq_block_step(n: Nat, d: Nat, k: Nat, q: Nat, j: Nat) {
    d != Nat.0 and d.divides(n) and divisor_quotient(n, d) = q
        and gcd_eq_block_pred(n, d, k, q)(j)
        implies gcd_eq_block_pred(n, d, k, q)(j.suc)
} by {
    if d != Nat.0 and d.divides(n) and divisor_quotient(n, d) = q
        and gcd_eq_block_pred(n, d, k, q)(j) {
        if k.coprime(q) and j.suc <= d and Nat.0 < j.suc {
            if j = Nat.0 {
                // Stepping past d * k itself: gcd(d*k, n) = d iff k coprime to q.
                gcd_mul_coprime_iff(n, d, k)
                k.coprime(q) = ((d * k).gcd(n) = d)
                (d * k).gcd(n) = d
                count_gcd_eq_suc_yes(n, d, d * k)
                count_gcd_eq(n, d, (d * k).suc) = count_gcd_eq(n, d, d * k) + Nat.1
                d * k + Nat.1 = (d * k).suc
                count_gcd_eq(n, d, d * k + Nat.1) = count_gcd_eq(n, d, d * k) + Nat.1
            } else {
                // 0 < j < d: the intermediate index is not a multiple of d.
                j != Nat.0
                pos_of_ne_zero(j)
                Nat.0 < j
                j < j.suc
                lt_and_lte(j, j.suc, d)
                j < d
                gcd_eq_block_mid_no(n, d, k, j)
                not ((d * k + j).gcd(n) = d)
                count_gcd_eq_suc_no(n, d, d * k + j)
                count_gcd_eq(n, d, (d * k + j).suc) = count_gcd_eq(n, d, d * k + j)
                d * k + j.suc = (d * k + j).suc
                count_gcd_eq(n, d, d * k + j.suc) = count_gcd_eq(n, d, d * k + j)
                gcd_eq_block_pred(n, d, k, q)(j)
                (k.coprime(q) and j <= d and Nat.0 < j) implies count_gcd_eq(n, d, d * k + j) = count_gcd_eq(n, d, d * k) + Nat.1
                j <= d
                Nat.0 < j
                count_gcd_eq(n, d, d * k + j) = count_gcd_eq(n, d, d * k) + Nat.1
                count_gcd_eq(n, d, d * k + j.suc) = count_gcd_eq(n, d, d * k) + Nat.1
            }
            count_gcd_eq(n, d, d * k + j.suc) = count_gcd_eq(n, d, d * k) + Nat.1
        }
        gcd_eq_block_pred(n, d, k, q)(j.suc) =
            (k.coprime(q) and j.suc <= d and Nat.0 < j.suc implies count_gcd_eq(n, d, d * k + j.suc) = count_gcd_eq(n, d, d * k) + Nat.1)
        gcd_eq_block_pred(n, d, k, q)(j.suc)
    }
}

/// The endpoint of the aligned block, coprime case: with `k` coprime to
/// `q = n / d`, the block `[d * k, d * k + d)` adds exactly one gcd-`d` element.
theorem gcd_eq_block(n: Nat, d: Nat, k: Nat, q: Nat) {
    d != Nat.0 and d.divides(n) and divisor_quotient(n, d) = q and k.coprime(q)
        implies count_gcd_eq(n, d, d * k + d) = count_gcd_eq(n, d, d * k) + Nat.1
} by {
    if d != Nat.0 and d.divides(n) and divisor_quotient(n, d) = q and k.coprime(q) {
        define g(x: Nat) -> Bool {
            x <= d and Nat.0 < x implies count_gcd_eq(n, d, d * k + x) = count_gcd_eq(n, d, d * k) + Nat.1
        }
        forall(y: Nat) {
            gcd_eq_block_pred(n, d, k, q)(y) = g(y)
            g(y) = gcd_eq_block_pred(n, d, k, q)(y)
        }
        if Nat.0 <= d and Nat.0 < Nat.0 {
            false
        }
        g(Nat.0)
        forall(x: Nat) {
            if g(x) {
                gcd_eq_block_pred(n, d, k, q)(x)
                gcd_eq_block_step(n, d, k, q, x)
                gcd_eq_block_pred(n, d, k, q)(x.suc)
                g(x.suc)
            }
        }
        forall(x: Nat) {
            g(x) implies g(x.suc)
        }
        g(Nat.0) and forall(x: Nat) {
            g(x) implies g(x.suc)
        }
        Nat.induction(g)
        Nat.0 < d
        d <= d
        g(d)
        g(d) = (d <= d and Nat.0 < d implies count_gcd_eq(n, d, d * k + d) = count_gcd_eq(n, d, d * k) + Nat.1)
        d <= d and Nat.0 < d implies count_gcd_eq(n, d, d * k + d) = count_gcd_eq(n, d, d * k) + Nat.1
        count_gcd_eq(n, d, d * k + d) = count_gcd_eq(n, d, d * k) + Nat.1
    }
}

/// Inductive predicate for the aligned-block walk, non-coprime case: with `k`
/// not coprime to `n / d`, none of the `d` indices after `d * k` has gcd `d`.
define gcd_eq_block_no_pred(n: Nat, d: Nat, k: Nat, q: Nat) -> (Nat -> Bool) {
    function(j: Nat) {
        j <= d and Nat.0 < j
            implies count_gcd_eq(n, d, d * k + j) = count_gcd_eq(n, d, d * k)
    }
}

/// Inductive step for the aligned-block walk, non-coprime case.
theorem gcd_eq_block_no_step(n: Nat, d: Nat, k: Nat, q: Nat, j: Nat) {
    d != Nat.0 and d.divides(n) and divisor_quotient(n, d) = q
        and not k.coprime(q) and gcd_eq_block_no_pred(n, d, k, q)(j)
        implies gcd_eq_block_no_pred(n, d, k, q)(j.suc)
} by {
    if d != Nat.0 and d.divides(n) and divisor_quotient(n, d) = q
        and not k.coprime(q) and gcd_eq_block_no_pred(n, d, k, q)(j) {
        if j.suc <= d and Nat.0 < j.suc {
            if j = Nat.0 {
                // Stepping past d * k: gcd(d*k, n) != d since k is not coprime.
                gcd_mul_coprime_iff(n, d, k)
                k.coprime(q) = ((d * k).gcd(n) = d)
                not ((d * k).gcd(n) = d)
                count_gcd_eq_suc_no(n, d, d * k)
                count_gcd_eq(n, d, (d * k).suc) = count_gcd_eq(n, d, d * k)
                d * k + Nat.1 = (d * k).suc
                count_gcd_eq(n, d, d * k + Nat.1) = count_gcd_eq(n, d, d * k)
            } else {
                j != Nat.0
                pos_of_ne_zero(j)
                Nat.0 < j
                j < j.suc
                lt_and_lte(j, j.suc, d)
                j < d
                gcd_eq_block_mid_no(n, d, k, j)
                not ((d * k + j).gcd(n) = d)
                count_gcd_eq_suc_no(n, d, d * k + j)
                count_gcd_eq(n, d, (d * k + j).suc) = count_gcd_eq(n, d, d * k + j)
                d * k + j.suc = (d * k + j).suc
                count_gcd_eq(n, d, d * k + j.suc) = count_gcd_eq(n, d, d * k + j)
                gcd_eq_block_no_pred(n, d, k, q)(j)
                j <= d and Nat.0 < j implies count_gcd_eq(n, d, d * k + j) = count_gcd_eq(n, d, d * k)
                j <= d
                Nat.0 < j
                count_gcd_eq(n, d, d * k + j) = count_gcd_eq(n, d, d * k)
                count_gcd_eq(n, d, d * k + j.suc) = count_gcd_eq(n, d, d * k)
            }
            count_gcd_eq(n, d, d * k + j.suc) = count_gcd_eq(n, d, d * k)
        }
        j.suc != Nat.0
        j.suc <= d and Nat.0 < j.suc implies count_gcd_eq(n, d, d * k + j.suc) = count_gcd_eq(n, d, d * k)
        gcd_eq_block_no_pred(n, d, k, q)(j.suc) =
            (j.suc <= d and Nat.0 < j.suc implies count_gcd_eq(n, d, d * k + j.suc) = count_gcd_eq(n, d, d * k))
        gcd_eq_block_no_pred(n, d, k, q)(j.suc)
    }
}

/// The endpoint of the aligned block, non-coprime case: with `k` not coprime
/// to `q = n / d`, the block `[d * k, d * k + d)` adds no gcd-`d` element.
theorem gcd_eq_block_no(n: Nat, d: Nat, k: Nat, q: Nat) {
    d != Nat.0 and d.divides(n) and divisor_quotient(n, d) = q and not k.coprime(q)
        implies count_gcd_eq(n, d, d * k + d) = count_gcd_eq(n, d, d * k)
} by {
    if d != Nat.0 and d.divides(n) and divisor_quotient(n, d) = q and not k.coprime(q) {
        define g(x: Nat) -> Bool {
            x <= d and Nat.0 < x implies count_gcd_eq(n, d, d * k + x) = count_gcd_eq(n, d, d * k)
        }
        forall(y: Nat) {
            gcd_eq_block_no_pred(n, d, k, q)(y) = g(y)
            g(y) = gcd_eq_block_no_pred(n, d, k, q)(y)
        }
        if Nat.0 <= d and Nat.0 < Nat.0 {
            false
        }
        g(Nat.0)
        forall(x: Nat) {
            if g(x) {
                gcd_eq_block_no_pred(n, d, k, q)(x)
                gcd_eq_block_no_step(n, d, k, q, x)
                gcd_eq_block_no_pred(n, d, k, q)(x.suc)
                g(x.suc)
            }
        }
        forall(x: Nat) {
            g(x) implies g(x.suc)
        }
        g(Nat.0) and forall(x: Nat) {
            g(x) implies g(x.suc)
        }
        Nat.induction(g)
        Nat.0 < d
        d <= d
        g(d)
        g(d) = (d <= d and Nat.0 < d implies count_gcd_eq(n, d, d * k + d) = count_gcd_eq(n, d, d * k))
        d <= d and Nat.0 < d implies count_gcd_eq(n, d, d * k + d) = count_gcd_eq(n, d, d * k)
        count_gcd_eq(n, d, d * k + d) = count_gcd_eq(n, d, d * k)
    }
}

/// Inductive predicate for `gcd_eq_quotient_count`: at `d * k <= n`, the
/// `gcd = d` fiber count over `[0, d * k)` equals the coprime count over
/// `[0, k)` with modulus `n / d`.
define gcd_eq_quotient_pred(n: Nat, d: Nat) -> (Nat -> Bool) {
    function(k: Nat) {
        d * k <= n implies count_gcd_eq(n, d, d * k) = count_coprime_to(divisor_quotient(n, d), k)
    }
}

/// Inductive step for `gcd_eq_quotient_pred`.
theorem gcd_eq_quotient_step(n: Nat, d: Nat, k: Nat) {
    d != Nat.0 and d.divides(n) and gcd_eq_quotient_pred(n, d)(k)
        implies gcd_eq_quotient_pred(n, d)(k.suc)
} by {
    if d != Nat.0 and d.divides(n) and gcd_eq_quotient_pred(n, d)(k) {
        if d * k.suc <= n {
            divisor_quotient_cofactor(n, d)
            d * divisor_quotient(n, d) = n
            d * k <= d * k.suc
            lte_trans(d * k, d * k.suc, n)
            d * k <= n
            count_gcd_eq(n, d, d * k) = count_coprime_to(divisor_quotient(n, d), k)
            d * k.suc = d * k + d
            if k.coprime(divisor_quotient(n, d)) {
                gcd_eq_block(n, d, k, divisor_quotient(n, d))
                count_gcd_eq(n, d, d * k + d) = count_gcd_eq(n, d, d * k) + Nat.1
                count_gcd_eq(n, d, d * k.suc) = count_gcd_eq(n, d, d * k) + Nat.1
                count_gcd_eq(n, d, d * k.suc) = count_coprime_to(divisor_quotient(n, d), k) + Nat.1
                count_coprime_to_suc_yes(divisor_quotient(n, d), k)
                count_coprime_to(divisor_quotient(n, d), k.suc) = count_coprime_to(divisor_quotient(n, d), k) + Nat.1
                count_gcd_eq(n, d, d * k.suc) = count_coprime_to(divisor_quotient(n, d), k.suc)
            } else {
                gcd_eq_block_no(n, d, k, divisor_quotient(n, d))
                count_gcd_eq(n, d, d * k + d) = count_gcd_eq(n, d, d * k)
                count_gcd_eq(n, d, d * k.suc) = count_gcd_eq(n, d, d * k)
                count_gcd_eq(n, d, d * k.suc) = count_coprime_to(divisor_quotient(n, d), k)
                count_coprime_to_suc_no(divisor_quotient(n, d), k)
                count_coprime_to(divisor_quotient(n, d), k.suc) = count_coprime_to(divisor_quotient(n, d), k)
                count_gcd_eq(n, d, d * k.suc) = count_coprime_to(divisor_quotient(n, d), k.suc)
            }
            count_gcd_eq(n, d, d * k.suc) = count_coprime_to(divisor_quotient(n, d), k.suc)
        }
        gcd_eq_quotient_pred(n, d)(k.suc)
    }
}

/// Walking the scaled count: `count_gcd_eq(n, d, d * k) = count_coprime_to(n / d, k)`.
theorem gcd_eq_quotient_count(n: Nat, d: Nat, k: Nat) {
    d != Nat.0 and d.divides(n) implies gcd_eq_quotient_pred(n, d)(k)
} by {
    if d != Nat.0 and d.divides(n) {
        define f(x: Nat) -> Bool {
            d * x <= n implies count_gcd_eq(n, d, d * x) = count_coprime_to(divisor_quotient(n, d), x)
        }
        forall(y: Nat) {
            gcd_eq_quotient_pred(n, d)(y) = f(y)
            f(y) = gcd_eq_quotient_pred(n, d)(y)
        }
        if d * Nat.0 <= n {
            d * Nat.0 = Nat.0
            count_gcd_eq(n, d, d * Nat.0) = count_gcd_eq(n, d, Nat.0)
            count_gcd_eq(n, d, Nat.0) = Nat.0
            count_coprime_to(divisor_quotient(n, d), Nat.0) = Nat.0
            count_gcd_eq(n, d, d * Nat.0) = count_coprime_to(divisor_quotient(n, d), Nat.0)
        }
        f(Nat.0)
        forall(x: Nat) {
            if f(x) {
                gcd_eq_quotient_pred(n, d)(x)
                gcd_eq_quotient_step(n, d, x)
                gcd_eq_quotient_pred(n, d)(x.suc)
                f(x.suc)
            }
        }
        forall(x: Nat) {
            f(x) implies f(x.suc)
        }
        f(Nat.0) and forall(x: Nat) {
            f(x) implies f(x.suc)
        }
        Nat.induction(f)
        f(k)
        gcd_eq_quotient_pred(n, d)(k)
    }
}

/// The fiber count over `[0, n)` of `gcd(x, n) = d` is `totient(n / d)`.
theorem count_gcd_eq_totient_quotient(n: Nat, d: Nat) {
    Nat.0 < n and d.divides(n)
        implies count_gcd_eq(n, d, n) = (divisor_quotient(n, d)).totient
} by {
    if Nat.0 < n and d.divides(n) {
        divisor_quotient_cofactor(n, d)
        d * divisor_quotient(n, d) = n
        if d = Nat.0 {
            Nat.0 * divisor_quotient(n, d) = Nat.0
            d * divisor_quotient(n, d) = Nat.0
            n = Nat.0
            false
        }
        d != Nat.0
        gcd_eq_quotient_count(n, d, divisor_quotient(n, d))
        gcd_eq_quotient_pred(n, d)(divisor_quotient(n, d))
        d * divisor_quotient(n, d) <= n implies count_gcd_eq(n, d, d * divisor_quotient(n, d)) = count_coprime_to(divisor_quotient(n, d), divisor_quotient(n, d))
        d * divisor_quotient(n, d) <= n
        count_gcd_eq(n, d, d * divisor_quotient(n, d)) = count_coprime_to(divisor_quotient(n, d), divisor_quotient(n, d))
        count_gcd_eq(n, d, n) = count_coprime_to(divisor_quotient(n, d), divisor_quotient(n, d))
        (divisor_quotient(n, d)).totient = count_coprime_to(divisor_quotient(n, d), divisor_quotient(n, d))
        count_gcd_eq(n, d, n) = (divisor_quotient(n, d)).totient
    }
}

/// For a bound `k < d`, no index in `[1, k]` can have gcd equal to `d`.
theorem count_gcd_eq_le_beyond(n: Nat, d: Nat, k: Nat) {
    k < d implies count_gcd_eq_le(n, d, k) = Nat.0
} by {
    define f(x: Nat) -> Bool {
        x < d implies count_gcd_eq_le(n, d, x) = Nat.0
    }
    if Nat.0 < d {
        count_gcd_eq_le(n, d, Nat.0) = Nat.0
    }
    f(Nat.0)
    forall(x: Nat) {
        if f(x) {
            if x.suc < d {
                x < x.suc
                lt_and_lte(x, x.suc, d)
                x < d
                count_gcd_eq_le(n, d, x) = Nat.0
                gcd_divides_left(x.suc, n)
                x.suc.gcd(n).divides(x.suc)
                x.suc != Nat.0
                divides_lte(x.suc.gcd(n), x.suc)
                x.suc.gcd(n) <= x.suc
                lte_and_lt(x.suc.gcd(n), x.suc, d)
                x.suc.gcd(n) < d
                not (x.suc.gcd(n) = d)
                count_gcd_eq_le_suc_no(n, d, x)
                count_gcd_eq_le(n, d, x.suc) = count_gcd_eq_le(n, d, x)
                count_gcd_eq_le(n, d, x.suc) = Nat.0
            }
            f(x.suc)
        }
    }
    forall(x: Nat) {
        f(x) implies f(x.suc)
    }
    f(Nat.0) and forall(x: Nat) {
        f(x) implies f(x.suc)
    }
    Nat.induction(f)
    if k < d {
        f(k)
        k < d implies count_gcd_eq_le(n, d, k) = Nat.0
        count_gcd_eq_le(n, d, k) = Nat.0
    }
}

/// Stepping the closed-range count: the increment is the indicator of
/// `gcd(k.suc, n) = d`.
theorem count_gcd_eq_le_suc_step(n: Nat, d: Nat, k: Nat) {
    count_gcd_eq_le(n, d, k.suc) =
        count_gcd_eq_le(n, d, k) + nat_indicator_eq(k.suc.gcd(n), d)
} by {
    if k.suc.gcd(n) = d {
        nat_indicator_eq(k.suc.gcd(n), d) = Nat.1
        count_gcd_eq_le_suc_yes(n, d, k)
        count_gcd_eq_le(n, d, k.suc) = count_gcd_eq_le(n, d, k) + Nat.1
        count_gcd_eq_le(n, d, k.suc) = count_gcd_eq_le(n, d, k) + nat_indicator_eq(k.suc.gcd(n), d)
    } else {
        nat_indicator_eq(k.suc.gcd(n), d) = Nat.0
        count_gcd_eq_le_suc_no(n, d, k)
        count_gcd_eq_le(n, d, k.suc) = count_gcd_eq_le(n, d, k)
        count_gcd_eq_le(n, d, k.suc) = count_gcd_eq_le(n, d, k) + nat_indicator_eq(k.suc.gcd(n), d)
    }
}

/// The point function of the equality indicator.
define ind_eq_fn(g: Nat) -> (Nat -> Nat) {
    function(x: Nat) { nat_indicator_eq(x, g) }
}

/// Stepping the open-range count: the increment is the indicator of
/// `gcd(k, n) = d` (the element added is `k`).
theorem count_gcd_eq_suc_step(n: Nat, d: Nat, k: Nat) {
    count_gcd_eq(n, d, k.suc) = count_gcd_eq(n, d, k) + nat_indicator_eq(k.gcd(n), d)
} by {
    if k.gcd(n) = d {
        nat_indicator_eq(k.gcd(n), d) = Nat.1
        count_gcd_eq_suc_yes(n, d, k)
        count_gcd_eq(n, d, k.suc) = count_gcd_eq(n, d, k) + Nat.1
        count_gcd_eq(n, d, k.suc) = count_gcd_eq(n, d, k) + nat_indicator_eq(k.gcd(n), d)
    } else {
        nat_indicator_eq(k.gcd(n), d) = Nat.0
        count_gcd_eq_suc_no(n, d, k)
        count_gcd_eq(n, d, k.suc) = count_gcd_eq(n, d, k)
        count_gcd_eq(n, d, k.suc) = count_gcd_eq(n, d, k) + nat_indicator_eq(k.gcd(n), d)
    }
}


/// Summing the indicator `[x = g]` over a list gives the count of `g`.
theorem sum_nat_indicator_eq_count(l: List[Nat], g: Nat) {
    sum(map(l, ind_eq_fn(g))) = l.count(g)
} by {
    define p(xs: List[Nat]) -> Bool {
        sum(map(xs, ind_eq_fn(g))) = xs.count(g)
    }
    map(List.nil[Nat], ind_eq_fn(g)) = List.nil[Nat]
    sum(List.nil[Nat]) = Nat.0
    List.nil[Nat].count(g) = Nat.0
    p(List.nil[Nat])
    forall(head: Nat, tail: List[Nat]) {
        if p(tail) {
            sum(map(tail, ind_eq_fn(g))) = tail.count(g)
            map(List.cons(head, tail), ind_eq_fn(g)) =
                List.cons(ind_eq_fn(g)(head), map(tail, ind_eq_fn(g)))
            ind_eq_fn(g)(head) = nat_indicator_eq(head, g)
            map(List.cons(head, tail), ind_eq_fn(g)) =
                List.cons(nat_indicator_eq(head, g), map(tail, ind_eq_fn(g)))
            sum(List.cons(nat_indicator_eq(head, g), map(tail, ind_eq_fn(g)))) =
                nat_indicator_eq(head, g) + sum(map(tail, ind_eq_fn(g)))
            sum(map(List.cons(head, tail), ind_eq_fn(g))) =
                nat_indicator_eq(head, g) + tail.count(g)
            if head = g {
                nat_indicator_eq(head, g) = Nat.1
                List.cons(head, tail).count(g) = Nat.1 + tail.count(g)
                sum(map(List.cons(head, tail), ind_eq_fn(g))) =
                    Nat.1 + tail.count(g)
                sum(map(List.cons(head, tail), ind_eq_fn(g))) =
                    List.cons(head, tail).count(g)
            } else {
                nat_indicator_eq(head, g) = Nat.0
                List.cons(head, tail).count(g) = tail.count(g)
                sum(map(List.cons(head, tail), ind_eq_fn(g))) =
                    Nat.0 + tail.count(g)
                Nat.0 + tail.count(g) = tail.count(g)
                sum(map(List.cons(head, tail), ind_eq_fn(g))) =
                    List.cons(head, tail).count(g)
            }
            sum(map(List.cons(head, tail), ind_eq_fn(g))) =
                List.cons(head, tail).count(g)
            p(List.cons(head, tail))
        }
    }
    forall(head: Nat, tail: List[Nat]) {
        p(tail) implies p(List.cons(head, tail))
    }
    p(List.nil[Nat]) and forall(head: Nat, tail: List[Nat]) {
        p(tail) implies p(List.cons(head, tail))
    }
    List.induction(p)
    forall(xs: List[Nat]) { p(xs) }
    p(l)
}

/// If `k` divides `n`, the gcd of `k` and `n` is `k` itself.
theorem gcd_self_divisor(k: Nat, n: Nat) {
    k.divides(n) implies k.gcd(n) = k
} by {
    if k.divides(n) {
        gcd_divides_left(k, n)
        k.gcd(n).divides(k)
        divides_self(k)
        k.divides(k)
        divides_gcd(k, k, n)
        k.divides(k.gcd(n))
        divides_symm(k.gcd(n), k)
        k.gcd(n) = k
    }
}

/// No element above the bound `k` occurs in `divisors_up_to(n, k)`.
theorem divisors_up_to_not_contains_gt(n: Nat, k: Nat, x: Nat) {
    k < x implies not divisors_up_to(n, k).contains(x)
} by {
    if k < x {
        if divisors_up_to(n, k).contains(x) {
            divisors_up_to_member(n, k, x)
            Nat.0 < x and x <= k and x.divides(n)
            x <= k
            lte_and_lt(x, k, x)
            x < x
            false
        }
        not divisors_up_to(n, k).contains(x)
    }
}

/// For `k.suc | n`, the closed-range fiber of `k.suc` at bound `k.suc` is `1`.
theorem count_gcd_eq_le_self_divisor(n: Nat, k: Nat) {
    k.suc.divides(n) implies count_gcd_eq_le(n, k.suc, k.suc) = Nat.1
} by {
    if k.suc.divides(n) {
        k < k.suc
        count_gcd_eq_le_beyond(n, k.suc, k)
        count_gcd_eq_le(n, k.suc, k) = Nat.0
        gcd_self_divisor(k.suc, n)
        k.suc.gcd(n) = k.suc
        count_gcd_eq_le_suc_yes(n, k.suc, k)
        count_gcd_eq_le(n, k.suc, k.suc) = count_gcd_eq_le(n, k.suc, k) + Nat.1
        count_gcd_eq_le(n, k.suc, k.suc) = Nat.0 + Nat.1
        Nat.0 + Nat.1 = Nat.1
        count_gcd_eq_le(n, k.suc, k.suc) = Nat.1
    }
}

/// The closed-range fiber count `[1, k]` and the open-range fiber count
/// `[0, k)` differ only by the two endpoints, whose indicators agree.
theorem gcd_count_shift(n: Nat, d: Nat, k: Nat) {
    count_gcd_eq_le(n, d, k) + nat_indicator_eq(n, d) =
        count_gcd_eq(n, d, k) + nat_indicator_eq(k.gcd(n), d)
} by {
    define f(x: Nat) -> Bool {
        count_gcd_eq_le(n, d, x) + nat_indicator_eq(n, d) =
            count_gcd_eq(n, d, x) + nat_indicator_eq(x.gcd(n), d)
    }
    gcd_zero_left(n)
    Nat.0.gcd(n) = n
    nat_indicator_eq(Nat.0.gcd(n), d) = nat_indicator_eq(n, d)
    count_gcd_eq_le(n, d, Nat.0) = Nat.0
    count_gcd_eq(n, d, Nat.0) = Nat.0
    count_gcd_eq_le(n, d, Nat.0) + nat_indicator_eq(n, d) =
        count_gcd_eq(n, d, Nat.0) + nat_indicator_eq(Nat.0.gcd(n), d)
    f(Nat.0)
    forall(x: Nat) {
        if f(x) {
            count_gcd_eq_le(n, d, x) + nat_indicator_eq(n, d) =
                count_gcd_eq(n, d, x) + nat_indicator_eq(x.gcd(n), d)
            count_gcd_eq_le_suc_step(n, d, x)
            count_gcd_eq_le(n, d, x.suc) = count_gcd_eq_le(n, d, x) + nat_indicator_eq(x.suc.gcd(n), d)
            count_gcd_eq_suc_step(n, d, x)
            count_gcd_eq(n, d, x.suc) = count_gcd_eq(n, d, x) + nat_indicator_eq(x.gcd(n), d)
            count_gcd_eq_le(n, d, x.suc) + nat_indicator_eq(n, d) =
                (count_gcd_eq_le(n, d, x) + nat_indicator_eq(x.suc.gcd(n), d)) + nat_indicator_eq(n, d)
            (count_gcd_eq_le(n, d, x) + nat_indicator_eq(x.suc.gcd(n), d)) + nat_indicator_eq(n, d) =
                count_gcd_eq_le(n, d, x) + (nat_indicator_eq(x.suc.gcd(n), d) + nat_indicator_eq(n, d))
            add_comm(nat_indicator_eq(x.suc.gcd(n), d), nat_indicator_eq(n, d))
            nat_indicator_eq(x.suc.gcd(n), d) + nat_indicator_eq(n, d) =
                nat_indicator_eq(n, d) + nat_indicator_eq(x.suc.gcd(n), d)
            count_gcd_eq_le(n, d, x) + (nat_indicator_eq(x.suc.gcd(n), d) + nat_indicator_eq(n, d)) =
                count_gcd_eq_le(n, d, x) + (nat_indicator_eq(n, d) + nat_indicator_eq(x.suc.gcd(n), d))
            count_gcd_eq_le(n, d, x.suc) + nat_indicator_eq(n, d) =
                count_gcd_eq_le(n, d, x) + (nat_indicator_eq(n, d) + nat_indicator_eq(x.suc.gcd(n), d))
            add_assoc(count_gcd_eq_le(n, d, x), nat_indicator_eq(n, d), nat_indicator_eq(x.suc.gcd(n), d))
            count_gcd_eq_le(n, d, x) + (nat_indicator_eq(n, d) + nat_indicator_eq(x.suc.gcd(n), d)) =
                (count_gcd_eq_le(n, d, x) + nat_indicator_eq(n, d)) + nat_indicator_eq(x.suc.gcd(n), d)
            count_gcd_eq_le(n, d, x.suc) + nat_indicator_eq(n, d) =
                (count_gcd_eq_le(n, d, x) + nat_indicator_eq(n, d)) + nat_indicator_eq(x.suc.gcd(n), d)
            (count_gcd_eq_le(n, d, x) + nat_indicator_eq(n, d)) + nat_indicator_eq(x.suc.gcd(n), d) =
                (count_gcd_eq(n, d, x) + nat_indicator_eq(x.gcd(n), d)) + nat_indicator_eq(x.suc.gcd(n), d)
            count_gcd_eq_le(n, d, x.suc) + nat_indicator_eq(n, d) =
                (count_gcd_eq(n, d, x) + nat_indicator_eq(x.gcd(n), d)) + nat_indicator_eq(x.suc.gcd(n), d)
            count_gcd_eq(n, d, x.suc) = count_gcd_eq(n, d, x) + nat_indicator_eq(x.gcd(n), d)
            count_gcd_eq(n, d, x.suc) + nat_indicator_eq(x.suc.gcd(n), d) =
                (count_gcd_eq(n, d, x) + nat_indicator_eq(x.gcd(n), d)) + nat_indicator_eq(x.suc.gcd(n), d)
            count_gcd_eq_le(n, d, x.suc) + nat_indicator_eq(n, d) =
                count_gcd_eq(n, d, x.suc) + nat_indicator_eq(x.suc.gcd(n), d)
            f(x.suc)
        }
    }
    forall(x: Nat) {
        f(x) implies f(x.suc)
    }
    f(Nat.0) and forall(x: Nat) {
        f(x) implies f(x.suc)
    }
    Nat.induction(f)
    f(k)
}

/// At the full bound `n`, the closed-range and open-range fiber counts agree.
theorem count_gcd_eq_le_eq_to(n: Nat, d: Nat) {
    count_gcd_eq_le(n, d, n) = count_gcd_eq(n, d, n)
} by {
    gcd_count_shift(n, d, n)
    count_gcd_eq_le(n, d, n) + nat_indicator_eq(n, d) =
        count_gcd_eq(n, d, n) + nat_indicator_eq(n.gcd(n), d)
    gcd_self_local(n)
    n.gcd(n) = n
    nat_indicator_eq(n.gcd(n), d) = nat_indicator_eq(n, d)
    count_gcd_eq_le(n, d, n) + nat_indicator_eq(n, d) =
        count_gcd_eq(n, d, n) + nat_indicator_eq(n, d)
    add_comm(count_gcd_eq_le(n, d, n), nat_indicator_eq(n, d))
    count_gcd_eq_le(n, d, n) + nat_indicator_eq(n, d) =
        nat_indicator_eq(n, d) + count_gcd_eq_le(n, d, n)
    add_comm(count_gcd_eq(n, d, n), nat_indicator_eq(n, d))
    count_gcd_eq(n, d, n) + nat_indicator_eq(n, d) =
        nat_indicator_eq(n, d) + count_gcd_eq(n, d, n)
    nat_indicator_eq(n, d) + count_gcd_eq_le(n, d, n) =
        nat_indicator_eq(n, d) + count_gcd_eq(n, d, n)
    add_cancels_left(nat_indicator_eq(n, d), count_gcd_eq_le(n, d, n), count_gcd_eq(n, d, n))
    count_gcd_eq_le(n, d, n) = count_gcd_eq(n, d, n)
}

/// The closed-range fiber count at a fixed bound, as a function of `d`.
define count_le_fn(n: Nat, k: Nat) -> (Nat -> Nat) {
    function(d: Nat) { count_gcd_eq_le(n, d, k) }
}

/// The indicator `[gcd(k.suc, n) = d]` as a function of `d`.
define gcd_le_ind_fn(n: Nat, k: Nat) -> (Nat -> Nat) {
    function(d: Nat) { nat_indicator_eq(k.suc.gcd(n), d) }
}

/// Stepping the bound in a sum over divisors: the sum increments by the count
/// of `gcd(k.suc, n)` in the list.
theorem partition_tail_suc_sum(n: Nat, k: Nat, l: List[Nat]) {
    sum(map(l, count_le_fn(n, k.suc))) =
        sum(map(l, count_le_fn(n, k))) + l.count(k.suc.gcd(n))
} by {
    forall(d: Nat) {
        if l.contains(d) {
            count_gcd_eq_le_suc_step(n, d, k)
            count_gcd_eq_le(n, d, k.suc) = count_gcd_eq_le(n, d, k) + nat_indicator_eq(k.suc.gcd(n), d)
            count_le_fn(n, k.suc)(d) = count_gcd_eq_le(n, d, k.suc)
            count_le_fn(n, k)(d) = count_gcd_eq_le(n, d, k)
            gcd_le_ind_fn(n, k)(d) = nat_indicator_eq(k.suc.gcd(n), d)
            add_fn[Nat, Nat](count_le_fn(n, k), gcd_le_ind_fn(n, k))(d) =
                count_le_fn(n, k)(d) + gcd_le_ind_fn(n, k)(d)
            count_le_fn(n, k.suc)(d) = add_fn[Nat, Nat](count_le_fn(n, k), gcd_le_ind_fn(n, k))(d)
        }
    }
    sum_map_of_pointwise(l, count_le_fn(n, k.suc),
        add_fn[Nat, Nat](count_le_fn(n, k), gcd_le_ind_fn(n, k)))
    sum(map(l, count_le_fn(n, k.suc))) =
        sum(map(l, add_fn[Nat, Nat](count_le_fn(n, k), gcd_le_ind_fn(n, k))))
    map_sum_add[Nat, Nat](l, count_le_fn(n, k), gcd_le_ind_fn(n, k))
    sum(map(l, count_le_fn(n, k))) + sum(map(l, gcd_le_ind_fn(n, k))) =
        sum(map(l, add_fn[Nat, Nat](count_le_fn(n, k), gcd_le_ind_fn(n, k))))
    forall(d: Nat) {
        if l.contains(d) {
            nat_indicator_eq_symm(k.suc.gcd(n), d)
            nat_indicator_eq(k.suc.gcd(n), d) = nat_indicator_eq(d, k.suc.gcd(n))
            gcd_le_ind_fn(n, k)(d) = nat_indicator_eq(k.suc.gcd(n), d)
            ind_eq_fn(k.suc.gcd(n))(d) = nat_indicator_eq(d, k.suc.gcd(n))
            gcd_le_ind_fn(n, k)(d) = ind_eq_fn(k.suc.gcd(n))(d)
        }
    }
    sum_map_of_pointwise(l, gcd_le_ind_fn(n, k), ind_eq_fn(k.suc.gcd(n)))
    sum(map(l, gcd_le_ind_fn(n, k))) = sum(map(l, ind_eq_fn(k.suc.gcd(n))))
    sum_nat_indicator_eq_count(l, k.suc.gcd(n))
    sum(map(l, ind_eq_fn(k.suc.gcd(n)))) = l.count(k.suc.gcd(n))
    sum(map(l, gcd_le_ind_fn(n, k))) = l.count(k.suc.gcd(n))
    sum(map(l, count_le_fn(n, k.suc))) =
        sum(map(l, count_le_fn(n, k))) + l.count(k.suc.gcd(n))
}

/// Inductive predicate for the divisor-sum partition: the closed-range fiber
/// counts over the divisors of `n` bounded by `k` add up to `k`.
define gcd_partition_pred(n: Nat) -> (Nat -> Bool) {
    function(k: Nat) {
        sum(map(divisors_up_to(n, k), count_le_fn(n, k))) = k
    }
}

/// A value at most another and not equal is smaller.
theorem lte_neq_imp_lt(a: Nat, b: Nat) {
    a <= b and a != b implies a < b
} by {
    if a <= b and a != b {
        lt_or_lte(a, b)
        if b <= a {
            lte_antisymm(a, b)
            a = b
            false
        }
        a < b
    }
}

/// A value smaller than `b.suc` is at most `b`.
theorem lt_suc_imp_lte(a: Nat, b: Nat) {
    a < b.suc implies a <= b
} by {
    if a < b.suc {
        lt_suc_right(a, b)
        if a = b {
            a <= b
        }
        if a < b {
            a <= b
        }
        a <= b
    }
}

/// Inductive step for `gcd_partition_pred`.
theorem gcd_partition_step(n: Nat, k: Nat) {
    Nat.0 < n and gcd_partition_pred(n)(k) implies gcd_partition_pred(n)(k.suc)
} by {
    if Nat.0 < n and gcd_partition_pred(n)(k) {
        sum(map(divisors_up_to(n, k), count_le_fn(n, k))) = k
        if k.suc.divides(n) {
            divisors_up_to_suc_yes(n, k)
            divisors_up_to(n, k.suc) = List.cons(k.suc, divisors_up_to(n, k))
            map(List.cons(k.suc, divisors_up_to(n, k)), count_le_fn(n, k.suc)) =
                List.cons(count_le_fn(n, k.suc)(k.suc),
                          map(divisors_up_to(n, k), count_le_fn(n, k.suc)))
            sum(List.cons(count_le_fn(n, k.suc)(k.suc),
                          map(divisors_up_to(n, k), count_le_fn(n, k.suc)))) =
                count_le_fn(n, k.suc)(k.suc) +
                sum(map(divisors_up_to(n, k), count_le_fn(n, k.suc)))
            sum(map(divisors_up_to(n, k.suc), count_le_fn(n, k.suc))) =
                count_le_fn(n, k.suc)(k.suc) +
                sum(map(divisors_up_to(n, k), count_le_fn(n, k.suc)))
            partition_tail_suc_sum(n, k, divisors_up_to(n, k))
            sum(map(divisors_up_to(n, k), count_le_fn(n, k.suc))) =
                sum(map(divisors_up_to(n, k), count_le_fn(n, k))) +
                divisors_up_to(n, k).count(k.suc.gcd(n))
            sum(map(divisors_up_to(n, k.suc), count_le_fn(n, k.suc))) =
                count_le_fn(n, k.suc)(k.suc) +
                (sum(map(divisors_up_to(n, k), count_le_fn(n, k))) +
                 divisors_up_to(n, k).count(k.suc.gcd(n)))
            count_gcd_eq_le_self_divisor(n, k)
            count_gcd_eq_le(n, k.suc, k.suc) = Nat.1
            count_le_fn(n, k.suc)(k.suc) = count_gcd_eq_le(n, k.suc, k.suc)
            count_le_fn(n, k.suc)(k.suc) = Nat.1
            gcd_self_divisor(k.suc, n)
            k.suc.gcd(n) = k.suc
            k < k.suc
            divisors_up_to_not_contains_gt(n, k, k.suc)
            not divisors_up_to(n, k).contains(k.suc)
            not_contains_count_zero(divisors_up_to(n, k), k.suc)
            divisors_up_to(n, k).count(k.suc) = Nat.0
            divisors_up_to(n, k).count(k.suc.gcd(n)) = Nat.0
            sum(map(divisors_up_to(n, k.suc), count_le_fn(n, k.suc))) =
                Nat.1 + (sum(map(divisors_up_to(n, k), count_le_fn(n, k))) + Nat.0)
            sum(map(divisors_up_to(n, k), count_le_fn(n, k))) + Nat.0 =
                sum(map(divisors_up_to(n, k), count_le_fn(n, k)))
            Nat.1 + (sum(map(divisors_up_to(n, k), count_le_fn(n, k))) + Nat.0) =
                Nat.1 + sum(map(divisors_up_to(n, k), count_le_fn(n, k)))
            sum(map(divisors_up_to(n, k.suc), count_le_fn(n, k.suc))) =
                Nat.1 + sum(map(divisors_up_to(n, k), count_le_fn(n, k)))
            sum(map(divisors_up_to(n, k.suc), count_le_fn(n, k.suc))) = Nat.1 + k
            Nat.1 + k = k.suc
            sum(map(divisors_up_to(n, k.suc), count_le_fn(n, k.suc))) = k.suc
        } else {
            divisors_up_to_suc_no(n, k)
            divisors_up_to(n, k.suc) = divisors_up_to(n, k)
            partition_tail_suc_sum(n, k, divisors_up_to(n, k))
            sum(map(divisors_up_to(n, k), count_le_fn(n, k.suc))) =
                sum(map(divisors_up_to(n, k), count_le_fn(n, k))) +
                divisors_up_to(n, k).count(k.suc.gcd(n))
            sum(map(divisors_up_to(n, k.suc), count_le_fn(n, k.suc))) =
                sum(map(divisors_up_to(n, k), count_le_fn(n, k))) +
                divisors_up_to(n, k).count(k.suc.gcd(n))
            gcd_divides_left(k.suc, n)
            k.suc.gcd(n).divides(k.suc)
            divides_lte(k.suc.gcd(n), k.suc)
            k.suc != Nat.0
            k.suc.gcd(n) <= k.suc
            if k.suc.gcd(n) = k.suc {
                gcd_divides_right(k.suc, n)
                k.suc.gcd(n).divides(n)
                k.suc.divides(n)
                false
            }
            k.suc.gcd(n) != k.suc
            lte_neq_imp_lt(k.suc.gcd(n), k.suc)
            k.suc.gcd(n) < k.suc
            lt_suc_imp_lte(k.suc.gcd(n), k)
            k.suc.gcd(n) <= k
            gcd_nonzero_left(k.suc, n)
            k.suc.gcd(n) != Nat.0
            pos_of_ne_zero(k.suc.gcd(n))
            Nat.0 < k.suc.gcd(n)
            gcd_divides_right(k.suc, n)
            k.suc.gcd(n).divides(n)
            divisors_up_to_complete(n, k, k.suc.gcd(n))
            divisors_up_to(n, k).contains(k.suc.gcd(n))
            divisors_up_to_unique(n, k)
            divisors_up_to(n, k).is_unique
            unique_count_one(divisors_up_to(n, k), k.suc.gcd(n))
            divisors_up_to(n, k).count(k.suc.gcd(n)) = Nat.1
            sum(map(divisors_up_to(n, k.suc), count_le_fn(n, k.suc))) =
                sum(map(divisors_up_to(n, k), count_le_fn(n, k))) + Nat.1
            sum(map(divisors_up_to(n, k.suc), count_le_fn(n, k.suc))) = k + Nat.1
            k + Nat.1 = k.suc
            sum(map(divisors_up_to(n, k.suc), count_le_fn(n, k.suc))) = k.suc
        }
        gcd_partition_pred(n)(k.suc) =
            (sum(map(divisors_up_to(n, k.suc), count_le_fn(n, k.suc))) = k.suc)
        gcd_partition_pred(n)(k.suc)
    }
}

/// Walking the divisor-sum partition up to `k`.
theorem gcd_partition_run(n: Nat, k: Nat) {
    Nat.0 < n implies gcd_partition_pred(n)(k)
} by {
    if Nat.0 < n {
        define f(x: Nat) -> Bool {
            sum(map(divisors_up_to(n, x), count_le_fn(n, x))) = x
        }
        forall(y: Nat) {
            gcd_partition_pred(n)(y) = f(y)
            f(y) = gcd_partition_pred(n)(y)
        }
        divisors_up_to_zero(n)
        divisors_up_to(n, Nat.0) = List.nil[Nat]
        map(List.nil[Nat], count_le_fn(n, Nat.0)) = List.nil[Nat]
        sum(List.nil[Nat]) = Nat.0
        f(Nat.0)
        forall(x: Nat) {
            if f(x) {
                gcd_partition_pred(n)(x)
                gcd_partition_step(n, x)
                gcd_partition_pred(n)(x.suc)
                f(x.suc)
            }
        }
        forall(x: Nat) {
            f(x) implies f(x.suc)
        }
        f(Nat.0) and forall(x: Nat) {
            f(x) implies f(x.suc)
        }
        Nat.induction(f)
        f(k)
        gcd_partition_pred(n)(k)
    }
}

/// The partition identity: `sum_{d | n} count_gcd_eq_le(n, d, n) = n`.
theorem gcd_partition(n: Nat) {
    Nat.0 < n implies
        sum(map(divisor_list(n), count_le_fn(n, n))) = n
} by {
    if Nat.0 < n {
        gcd_partition_run(n, n)
        gcd_partition_pred(n)(n)
        sum(map(divisors_up_to(n, n), count_le_fn(n, n))) = n
        divisor_list(n) = divisors_up_to(n, n)
        sum(map(divisor_list(n), count_le_fn(n, n))) = n
    }
}

/// The quotient map `d -> n / d` is a function on divisors.
define divisor_quotient_fn(n: Nat) -> (Nat -> Nat) {
    function(d: Nat) { divisor_quotient(n, d) }
}

/// Dividing twice by the same divisor restores the divisor.
theorem divisor_quotient_involution(n: Nat, d: Nat) {
    Nat.0 < n and d.divides(n)
        implies divisor_quotient(n, divisor_quotient(n, d)) = d
} by {
    if Nat.0 < n and d.divides(n) {
        divisor_quotient_cofactor(n, d)
        d * divisor_quotient(n, d) = n
        divisor_quotient_divides(n, d)
        divisor_quotient(n, d).divides(n)
        divisor_quotient_cofactor(n, divisor_quotient(n, d))
        divisor_quotient(n, d) * divisor_quotient(n, divisor_quotient(n, d)) = n
        divisor_quotient_positive(n, d)
        Nat.0 < divisor_quotient(n, d)
        divisor_quotient(n, d) != Nat.0
        mul_comm(d, divisor_quotient(n, d))
        d * divisor_quotient(n, d) = divisor_quotient(n, d) * d
        divisor_quotient(n, d) * d = n
        mul_cancel_left(divisor_quotient(n, d), d, divisor_quotient(n, divisor_quotient(n, d)))
        d = divisor_quotient(n, divisor_quotient(n, d))
        divisor_quotient(n, divisor_quotient(n, d)) = d
    }
}

/// The quotient map permutes the divisor list of `n`.
theorem divisor_quotient_map_permutation(n: Nat) {
    Nat.0 < n implies
        is_permutation(map(divisor_list(n), divisor_quotient_fn(n)), divisor_list(n))
} by {
    if Nat.0 < n {
        divisor_list_is_unique(n)
        divisor_list(n).is_unique
        forall(a: Nat, b: Nat) {
            if divisor_list(n).contains(a) and divisor_list(n).contains(b)
                and divisor_quotient_fn(n)(a) = divisor_quotient_fn(n)(b) {
                divisor_list_contains_implies(n, a)
                Nat.0 < a and a.divides(n)
                a.divides(n)
                divisor_list_contains_implies(n, b)
                b.divides(n)
                divisor_quotient_cofactor(n, a)
                a * divisor_quotient(n, a) = n
                divisor_quotient_cofactor(n, b)
                b * divisor_quotient(n, b) = n
                divisor_quotient_fn(n)(a) = divisor_quotient(n, a)
                divisor_quotient_fn(n)(b) = divisor_quotient(n, b)
                divisor_quotient(n, a) = divisor_quotient(n, b)
                divisor_quotient_positive(n, a)
                Nat.0 < divisor_quotient(n, a)
                divisor_quotient(n, a) != Nat.0
                mul_comm(a, divisor_quotient(n, a))
                a * divisor_quotient(n, a) = divisor_quotient(n, a) * a
                divisor_quotient(n, a) * a = n
                mul_comm(b, divisor_quotient(n, b))
                b * divisor_quotient(n, b) = divisor_quotient(n, b) * b
                divisor_quotient(n, b) * b = n
                divisor_quotient(n, a) * a = divisor_quotient(n, b) * b
                divisor_quotient(n, a) * a = divisor_quotient(n, a) * b
                mul_cancel_left(divisor_quotient(n, a), a, b)
                a = b
            }
        }
        locally_injective_map_is_unique(divisor_list(n), divisor_quotient_fn(n))
        map(divisor_list(n), divisor_quotient_fn(n)).is_unique
        forall(y: Nat) {
            if map(divisor_list(n), divisor_quotient_fn(n)).contains(y) {
                map_contains(divisor_list(n), divisor_quotient_fn(n), y)
                let a: Nat satisfy {
                    divisor_list(n).contains(a) and divisor_quotient_fn(n)(a) = y
                }
                divisor_list(n).contains(a)
                divisor_list_contains_implies(n, a)
                Nat.0 < a and a.divides(n)
                a.divides(n)
                divisor_quotient_divides(n, a)
                divisor_quotient(n, a).divides(n)
                divisor_quotient_positive(n, a)
                Nat.0 < divisor_quotient(n, a)
                divisor_list_contains_of(n, divisor_quotient(n, a))
                divisor_list(n).contains(divisor_quotient(n, a))
                divisor_quotient_fn(n)(a) = divisor_quotient(n, a)
                divisor_list(n).contains(y)
            }
            if divisor_list(n).contains(y) {
                divisor_list_contains_implies(n, y)
                Nat.0 < y and y.divides(n)
                y.divides(n)
                divisor_quotient_involution(n, y)
                divisor_quotient(n, divisor_quotient(n, y)) = y
                divisor_quotient_divides(n, y)
                divisor_quotient(n, y).divides(n)
                divisor_quotient_positive(n, y)
                Nat.0 < divisor_quotient(n, y)
                divisor_list_contains_of(n, divisor_quotient(n, y))
                divisor_list(n).contains(divisor_quotient(n, y))
                divisor_quotient_fn(n)(divisor_quotient(n, y)) = divisor_quotient(n, divisor_quotient(n, y))
                map_contains_of_contains(divisor_list(n), divisor_quotient_fn(n), divisor_quotient(n, y))
                map(divisor_list(n), divisor_quotient_fn(n)).contains(
                    divisor_quotient_fn(n)(divisor_quotient(n, y)))
                map(divisor_list(n), divisor_quotient_fn(n)).contains(y)
            }
            map(divisor_list(n), divisor_quotient_fn(n)).contains(y) = divisor_list(n).contains(y)
        }
        unique_same_contains_imp_permutation(
            map(divisor_list(n), divisor_quotient_fn(n)), divisor_list(n))
        is_permutation(map(divisor_list(n), divisor_quotient_fn(n)), divisor_list(n))
    }
}

/// Reindexing a divisor sum by `d -> n / d` leaves the sum unchanged.
theorem divisor_quotient_reindex(f: Nat -> Nat, n: Nat) {
    Nat.0 < n implies
        sum(map(divisor_list(n), function(d: Nat) { f(divisor_quotient(n, d)) })) =
        sum(map(divisor_list(n), f))
} by {
    if Nat.0 < n {
        divisor_quotient_map_permutation(n)
        is_permutation(map(divisor_list(n), divisor_quotient_fn(n)), divisor_list(n))
        permutation_preserves_mapped_sum(
            map(divisor_list(n), divisor_quotient_fn(n)), divisor_list(n), f)
        sum(map(map(divisor_list(n), divisor_quotient_fn(n)), f)) = sum(map(divisor_list(n), f))
        map_map(divisor_list(n), divisor_quotient_fn(n), f)
        map(map(divisor_list(n), divisor_quotient_fn(n)), f) =
            map(divisor_list(n), compose(f, divisor_quotient_fn(n)))
        compose(f, divisor_quotient_fn(n)) = function(d: Nat) { f(divisor_quotient_fn(n)(d)) }
        forall(d: Nat) {
            if divisor_list(n).contains(d) {
                divisor_quotient_fn(n)(d) = divisor_quotient(n, d)
                function(x: Nat) { f(divisor_quotient_fn(n)(x)) }(d) =
                    f(divisor_quotient_fn(n)(d))
                function(x: Nat) { f(divisor_quotient_fn(n)(x)) }(d) = f(divisor_quotient(n, d))
                function(x: Nat) { f(divisor_quotient(n, x)) }(d) = f(divisor_quotient(n, d))
                function(x: Nat) { f(divisor_quotient_fn(n)(x)) }(d) =
                    function(x: Nat) { f(divisor_quotient(n, x)) }(d)
            }
        }
        sum_map_of_pointwise(divisor_list(n),
            function(d: Nat) { f(divisor_quotient_fn(n)(d)) },
            function(d: Nat) { f(divisor_quotient(n, d)) })
        sum(map(divisor_list(n), function(d: Nat) { f(divisor_quotient_fn(n)(d)) })) =
            sum(map(divisor_list(n), function(d: Nat) { f(divisor_quotient(n, d)) }))
        map(divisor_list(n), compose(f, divisor_quotient_fn(n))) =
            map(divisor_list(n), function(d: Nat) { f(divisor_quotient_fn(n)(d)) })
        sum(map(divisor_list(n), compose(f, divisor_quotient_fn(n)))) =
            sum(map(divisor_list(n), function(d: Nat) { f(divisor_quotient(n, d)) }))
        sum(map(map(divisor_list(n), divisor_quotient_fn(n)), f)) =
            sum(map(divisor_list(n), function(d: Nat) { f(divisor_quotient(n, d)) }))
        sum(map(divisor_list(n), function(d: Nat) { f(divisor_quotient(n, d)) })) =
            sum(map(divisor_list(n), f))
    }
}

/// The classical summation identity: `sum_{d | n} totient(d) = n`.
///
/// The numbers `x` in `[1, n]` are partitioned by the value `gcd(x, n) = d`;
/// the fiber over `d` has `totient(n / d)` elements, and summing over the
/// divisors of `n` reindexed by `d -> n / d` yields `n`.
theorem totient_divisor_sum_identity(n: Nat) {
    divisor_sum_fn(nat_totient)(n) = n
} by {
    if n = Nat.0 {
        divisor_sum_fn_at_zero(nat_totient)
        divisor_sum_fn(nat_totient)(Nat.0) = Nat.0
        divisor_sum_fn(nat_totient)(n) = n
    } else {
        n != Nat.0
        Nat.0 < n
        divisor_sum_fn_apply(nat_totient, n)
        divisor_sum_fn(nat_totient)(n) = sum(map(divisor_list(n), nat_totient))
        divisor_quotient_reindex(nat_totient, n)
        sum(map(divisor_list(n), function(d: Nat) { nat_totient(divisor_quotient(n, d)) })) =
            sum(map(divisor_list(n), nat_totient))
        forall(d: Nat) {
            if divisor_list(n).contains(d) {
                divisor_list_contains_implies(n, d)
                Nat.0 < d and d.divides(n)
                d.divides(n)
                count_gcd_eq_totient_quotient(n, d)
                count_gcd_eq(n, d, n) = (divisor_quotient(n, d)).totient
                count_gcd_eq_le_eq_to(n, d)
                count_gcd_eq_le(n, d, n) = count_gcd_eq(n, d, n)
                (divisor_quotient(n, d)).totient = count_gcd_eq(n, d, n)
                (divisor_quotient(n, d)).totient = count_gcd_eq_le(n, d, n)
                nat_totient(divisor_quotient(n, d)) = (divisor_quotient(n, d)).totient
                nat_totient(divisor_quotient(n, d)) = count_gcd_eq_le(n, d, n)
            }
        }
        sum_map_of_pointwise(divisor_list(n),
            function(d: Nat) { nat_totient(divisor_quotient(n, d)) },
            function(d: Nat) { count_gcd_eq_le(n, d, n) })
        sum(map(divisor_list(n), function(d: Nat) { nat_totient(divisor_quotient(n, d)) })) =
            sum(map(divisor_list(n), function(d: Nat) { count_gcd_eq_le(n, d, n) }))
        gcd_partition(n)
        sum(map(divisor_list(n), count_le_fn(n, n))) = n
        forall(d: Nat) {
            count_le_fn(n, n)(d) = count_gcd_eq_le(n, d, n)
        }
        sum_map_of_pointwise(divisor_list(n), count_le_fn(n, n),
            function(d: Nat) { count_gcd_eq_le(n, d, n) })
        sum(map(divisor_list(n), count_le_fn(n, n))) =
            sum(map(divisor_list(n), function(d: Nat) { count_gcd_eq_le(n, d, n) }))
        divisor_sum_fn(nat_totient)(n) = n
    }
}

/// Möbius inversion for Euler's totient:
///   `totient(n) = sum_{d | n} mu(d) * (n / d)`.
///
/// BONUS target — not yet proved. The statement is the Int-valued inversion of
/// `totient_divisor_sum_identity` (sum_{d | n} totient(d) = n), specialised to
/// `f = nat_totient`:
///
///   sum_{d | n} mu(d) * (n / d)
///     = sum_{d | n} mu(d) * (sum_{e | n/d} totient(e))        [by (a) at n/d]
///     = sum_{e | n} totient(e) * (sum_{d | n/e} mu(d))         [interchange]
///     = sum_{e | n} totient(e) * [n/e = 1]                     [fundamental identity]
///     = totient(n).
///
/// The missing step is the interchange of the two divisor sums over the pairs
/// `(d, e)` with `d * e | n`, i.e. `is_permutation` between
/// `map(divisor_list(n), d -> map(divisor_list(n / d), e -> (d, e)))` flattened
/// and the same with the roles of `d` and `e` swapped.  This is exactly the
/// piece recorded as missing at the end of `mobius_inversion.ac`
/// (`mobius_inversion` is commented out there for the same reason).  The
/// statement below is the intended form; `Int.from_nat` lifts the natural
/// quotient `n / d` to the integers so the product with `mu(d)` is well-typed.
///
// theorem totient_mobius_inversion(n: Nat) {
//     Nat.0 < n implies
//         sum(map(divisor_list(n), function(d: Nat) {
//             nat_mobius(d) * Int.from_nat(divisor_quotient(n, d))
//         })) = Int.from_nat(n.totient)
// }
