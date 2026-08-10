// Carmichael numbers and the Carmichael function λ.
//
// This file builds on `carmichael.ac` (the definitions of `carmichael_witness`,
// `is_carmichael` and the Carmichael function `carmichael` = λ, together with
// the core facts `carmichael_divides_totient` and `carmichael_pow_congr_one`)
// and proves the classical layer on top:
//
//   (a) 561 = 3 · 11 · 17 is a Carmichael number: every unit `a` modulo 561
//       satisfies a^560 ≡ 1 (mod 561).  (561 is the smallest Carmichael
//       number, but the minimality statement is left as a comment below.)
//   (b) Korselt's criterion is stated (as a comment) at the end: the
//       general proof needs list-level CRT machinery over the prime
//       factorisation, which is not yet in the library.  The concrete
//       Korselt conditions for 561 are proved.
//   (c) λ(n) | φ(n) for every positive n (restated from `carmichael.ac`).
//   (d) λ(p^k) | φ(p^k) for prime powers; the equality λ(p^k) = φ(p^k) for
//       odd primes needs a primitive root modulo p^k (not yet in the library)
//       and is stated as a comment.
//   (e) a^λ(n) ≡ 1 (mod n) for every unit a (restated from `carmichael.ac`).

from number_theory.carmichael import Nat, carmichael_witness,
    carmichael_witness_apply, carmichael, carmichael_pow_congr_one,
    carmichael_divides_totient, three_is_prime, lt_one_two, lt_two_three,
    lt_one_four, one_ne_zero
from number_theory.totient import euler, totient_prime
from number_theory.congruence import congr_mod_pow, mod_add_mul
from number_theory.coprime import coprime_comm, coprime_mul, coprime_mul_iff
from number_theory.crt import nat_congr_combine_coprime
from number_theory.factorisation import coprime_of_distinct_primes
from number_theory.zsigmondy import lt_ne
from nat import exp_mul, one_exp, exp_ne_zero, lt_suc, lt_suc_right, lt_not_ref,
    not_lt_zero, lt_imp_lt_suc, lt_imp_lte_suc, lt_or_lte, lte_and_lt,
    lte_imp_not_lt, lte_trans, lt_add_suc, small_mod, div_imp_mod, mul_comm,
    mul_assoc, mul_one_left, mul_one_right, mul_zero_right,
    add_assoc, add_imp_sub, lte_mul_both, lt_mul_both, read_add_single, read_add_read,
    read_mul_single, read_read_carry, nat_mul_1_3, nat_mul_2_2, nat_mul_3_6,
    nat_mul_4_4, nat_mul_5_3, nat_mul_5_5, nat_mul_5_6, nat_mul_7_3,
    nat_mul_8_2, nat_mul_8_3
from number_theory.mersenne_perfect import read_pos
numerals Nat

// ---------------------------------------------------------------------------
// Small decimal-arithmetic facts.
//
// The automatic normaliser of Acorn handles only small numeral arithmetic, so
// the products needed for 561 = 3 · 11 · 17 and 560 = 2 · 280 = 10 · 56 =
// 16 · 35 are proved here digit-wise through the decimal `read` bridge.
// ---------------------------------------------------------------------------

/// `28 * 2 = 56`.
theorem nat_mul_28_2 {
    Nat.28 * Nat.2 = Nat.56
} by {
    Nat.28 = Nat.2.read(Nat.8)
    read_mul_single(Nat.2, Nat.8, Nat.2)
    Nat.2.read(Nat.8) * Nat.2 = (Nat.2 * Nat.2).read(Nat.8 * Nat.2)
    nat_mul_2_2
    Nat.2 * Nat.2 = Nat.4
    nat_mul_8_2
    Nat.8 * Nat.2 = Nat.16
    Nat.28 * Nat.2 = Nat.4.read(Nat.16)
    read_read_carry(Nat.4, Nat.1, Nat.6)
    Nat.4.read(Nat.16) = (Nat.4 + Nat.1).read(Nat.6)
    Nat.4 + Nat.1 = Nat.5
    Nat.28 * Nat.2 = Nat.5.read(Nat.6)
    Nat.5.read(Nat.6) = Nat.56
    Nat.28 * Nat.2 = Nat.56
}

/// `2 * 280 = 560`.
theorem nat_mul_2_280 {
    Nat.2 * Nat.280 = Nat.560
} by {
    Nat.280 = Nat.28.read(Nat.0)
    read_mul_single(Nat.28, Nat.0, Nat.2)
    Nat.28.read(Nat.0) * Nat.2 = (Nat.28 * Nat.2).read(Nat.0 * Nat.2)
    mul_comm(Nat.2, Nat.280)
    Nat.2 * Nat.280 = Nat.280 * Nat.2
    Nat.2 * Nat.280 = (Nat.28 * Nat.2).read(Nat.0 * Nat.2)
    nat_mul_28_2
    Nat.28 * Nat.2 = Nat.56
    mul_zero_right(Nat.2)
    Nat.0 * Nat.2 = Nat.0
    Nat.2 * Nat.280 = Nat.56.read(Nat.0)
    Nat.56.read(Nat.0) = Nat.560
    Nat.2 * Nat.280 = Nat.560
}

/// `18 * 3 = 54`.
theorem nat_mul_18_3 {
    Nat.18 * Nat.3 = Nat.54
} by {
    Nat.18 = Nat.1.read(Nat.8)
    read_mul_single(Nat.1, Nat.8, Nat.3)
    Nat.1.read(Nat.8) * Nat.3 = (Nat.1 * Nat.3).read(Nat.8 * Nat.3)
    nat_mul_1_3
    Nat.1 * Nat.3 = Nat.3
    nat_mul_8_3
    Nat.8 * Nat.3 = Nat.24
    Nat.18 * Nat.3 = Nat.3.read(Nat.24)
    read_read_carry(Nat.3, Nat.2, Nat.4)
    Nat.3.read(Nat.24) = (Nat.3 + Nat.2).read(Nat.4)
    Nat.3 + Nat.2 = Nat.5
    Nat.18 * Nat.3 = Nat.5.read(Nat.4)
    Nat.5.read(Nat.4) = Nat.54
    Nat.18 * Nat.3 = Nat.54
}

/// `3 * 187 = 561`.
theorem nat_mul_3_187 {
    Nat.3 * Nat.187 = Nat.561
} by {
    Nat.187 = Nat.18.read(Nat.7)
    read_mul_single(Nat.18, Nat.7, Nat.3)
    Nat.18.read(Nat.7) * Nat.3 = (Nat.18 * Nat.3).read(Nat.7 * Nat.3)
    mul_comm(Nat.3, Nat.187)
    Nat.3 * Nat.187 = Nat.187 * Nat.3
    Nat.3 * Nat.187 = (Nat.18 * Nat.3).read(Nat.7 * Nat.3)
    nat_mul_18_3
    Nat.18 * Nat.3 = Nat.54
    nat_mul_7_3
    Nat.7 * Nat.3 = Nat.21
    Nat.3 * Nat.187 = Nat.54.read(Nat.21)
    read_read_carry(Nat.54, Nat.2, Nat.1)
    Nat.54.read(Nat.21) = (Nat.54 + Nat.2).read(Nat.1)
    Nat.54 + Nat.2 = Nat.56
    Nat.3 * Nat.187 = Nat.56.read(Nat.1)
    Nat.56.read(Nat.1) = Nat.561
    Nat.3 * Nat.187 = Nat.561
}

/// `11 * 17 = 187`.
theorem nat_mul_11_17 {
    Nat.11 * Nat.17 = Nat.187
}

/// `17 - 1 = 16`.
theorem nat_sub_17_1 {
    Nat.17 - Nat.1 = Nat.16
} by {
    Nat.16 + Nat.1 = Nat.17
    add_imp_sub(Nat.16, Nat.1, Nat.17)
    Nat.16 + Nat.1 = Nat.17 implies Nat.17 - Nat.1 = Nat.16
    Nat.17 - Nat.1 = Nat.16
}

/// `3 * 11 * 17 = 561`.
theorem nat_three_eleven_seventeen {
    Nat.3 * Nat.11 * Nat.17 = Nat.561
} by {
    mul_assoc(Nat.3, Nat.11, Nat.17)
    Nat.3 * Nat.11 * Nat.17 = Nat.3 * (Nat.11 * Nat.17)
    nat_mul_11_17
    Nat.11 * Nat.17 = Nat.187
    Nat.3 * Nat.11 * Nat.17 = Nat.3 * Nat.187
    nat_mul_3_187
    Nat.3 * Nat.187 = Nat.561
    Nat.3 * Nat.11 * Nat.17 = Nat.561
}

/// `8 + 3 = 11`.
theorem nat_add_8_3_11 {
    Nat.8 + Nat.3 = Nat.11
} by {
    Nat.2 + Nat.1 = Nat.3
    add_assoc(Nat.8, Nat.2, Nat.1)
    (Nat.8 + Nat.2) + Nat.1 = Nat.8 + (Nat.2 + Nat.1)
    Nat.8 + Nat.3 = (Nat.8 + Nat.2) + Nat.1
    Nat.8 + Nat.2 = Nat.10
    Nat.8 + Nat.3 = Nat.10 + Nat.1
    Nat.10 + Nat.1 = Nat.11
    Nat.8 + Nat.3 = Nat.11
}

/// `7 + 8 = 15`.
theorem nat_add_7_8_15 {
    Nat.7 + Nat.8 = Nat.15
} by {
    Nat.7 + Nat.1 = Nat.8
    add_assoc(Nat.7, Nat.7, Nat.1)
    (Nat.7 + Nat.7) + Nat.1 = Nat.7 + (Nat.7 + Nat.1)
    Nat.7 + Nat.8 = (Nat.7 + Nat.7) + Nat.1
    Nat.7 + Nat.7 = Nat.14
    Nat.7 + Nat.8 = Nat.14 + Nat.1
    Nat.14 + Nat.1 = Nat.15
    Nat.7 + Nat.8 = Nat.15
}

/// `18 + 3 = 21`.
theorem nat_add_18_3 {
    Nat.18 + Nat.3 = Nat.21
} by {
    Nat.18 = Nat.1.read(Nat.8)
    read_add_single(Nat.1, Nat.8, Nat.3)
    Nat.1.read(Nat.8) + Nat.3 = Nat.1.read(Nat.8 + Nat.3)
    nat_add_8_3_11
    Nat.8 + Nat.3 = Nat.11
    Nat.18 + Nat.3 = Nat.1.read(Nat.11)
    Nat.1.read(Nat.11) = Nat.21
    Nat.18 + Nat.3 = Nat.21
}

/// `11 + 6 = 17`.
theorem nat_add_11_6 {
    Nat.11 + Nat.6 = Nat.17
} by {
    Nat.11 = Nat.1.read(Nat.1)
    read_add_single(Nat.1, Nat.1, Nat.6)
    Nat.1.read(Nat.1) + Nat.6 = Nat.1.read(Nat.1 + Nat.6)
    Nat.1 + Nat.6 = Nat.7
    Nat.11 + Nat.6 = Nat.1.read(Nat.7)
    Nat.1.read(Nat.7) = Nat.17
    Nat.11 + Nat.6 = Nat.17
}

/// `3 + 14 = 17`.
theorem nat_add_3_14 {
    Nat.3 + Nat.14 = Nat.17
} by {
    Nat.14 = Nat.1.read(Nat.4)
    read_add_read(Nat.0, Nat.3, Nat.1, Nat.4)
    Nat.0.read(Nat.3) + Nat.1.read(Nat.4) = (Nat.0 + Nat.1).read(Nat.3 + Nat.4)
    Nat.0 + Nat.1 = Nat.1
    Nat.3 + Nat.4 = Nat.7
    Nat.3 + Nat.14 = Nat.1.read(Nat.7)
    Nat.1.read(Nat.7) = Nat.17
    Nat.3 + Nat.14 = Nat.17
}

/// `6 * 35 = 210`.
theorem nat_mul_6_35 {
    Nat.6 * Nat.35 = Nat.210
} by {
    Nat.35 = Nat.3.read(Nat.5)
    read_mul_single(Nat.3, Nat.5, Nat.6)
    Nat.3.read(Nat.5) * Nat.6 = (Nat.3 * Nat.6).read(Nat.5 * Nat.6)
    mul_comm(Nat.6, Nat.35)
    Nat.6 * Nat.35 = Nat.35 * Nat.6
    Nat.6 * Nat.35 = (Nat.3 * Nat.6).read(Nat.5 * Nat.6)
    nat_mul_3_6
    Nat.3 * Nat.6 = Nat.18
    nat_mul_5_6
    Nat.5 * Nat.6 = Nat.30
    Nat.6 * Nat.35 = Nat.18.read(Nat.30)
    read_read_carry(Nat.18, Nat.3, Nat.0)
    Nat.18.read(Nat.30) = (Nat.18 + Nat.3).read(Nat.0)
    nat_add_18_3
    Nat.18 + Nat.3 = Nat.21
    Nat.6 * Nat.35 = Nat.21.read(Nat.0)
    Nat.21.read(Nat.0) = Nat.210
    Nat.6 * Nat.35 = Nat.210
}

/// `35 + 21 = 56`.
theorem nat_add_35_21 {
    Nat.35 + Nat.21 = Nat.56
} by {
    Nat.35 = Nat.3.read(Nat.5)
    Nat.21 = Nat.2.read(Nat.1)
    read_add_read(Nat.3, Nat.5, Nat.2, Nat.1)
    Nat.3.read(Nat.5) + Nat.2.read(Nat.1) = (Nat.3 + Nat.2).read(Nat.5 + Nat.1)
    Nat.3 + Nat.2 = Nat.5
    Nat.5 + Nat.1 = Nat.6
    Nat.35 + Nat.21 = Nat.5.read(Nat.6)
    Nat.5.read(Nat.6) = Nat.56
    Nat.35 + Nat.21 = Nat.56
}

/// `16 * 35 = 560`.
theorem nat_mul_16_35 {
    Nat.16 * Nat.35 = Nat.560
} by {
    Nat.16 = Nat.1.read(Nat.6)
    read_mul_single(Nat.1, Nat.6, Nat.35)
    Nat.1.read(Nat.6) * Nat.35 = (Nat.1 * Nat.35).read(Nat.6 * Nat.35)
    mul_one_left(Nat.35)
    Nat.1 * Nat.35 = Nat.35
    nat_mul_6_35
    Nat.6 * Nat.35 = Nat.210
    Nat.16 * Nat.35 = Nat.35.read(Nat.210)
    read_read_carry(Nat.35, Nat.21, Nat.0)
    Nat.35.read(Nat.210) = (Nat.35 + Nat.21).read(Nat.0)
    nat_add_35_21
    Nat.35 + Nat.21 = Nat.56
    Nat.16 * Nat.35 = Nat.56.read(Nat.0)
    Nat.56.read(Nat.0) = Nat.560
    Nat.16 * Nat.35 = Nat.560
}

/// `15 + 2 = 17`.
theorem nat_add_15_2 {
    Nat.15 + Nat.2 = Nat.17
} by {
    Nat.15 = Nat.1.read(Nat.5)
    read_add_single(Nat.1, Nat.5, Nat.2)
    Nat.1.read(Nat.5) + Nat.2 = Nat.1.read(Nat.5 + Nat.2)
    Nat.5 + Nat.2 = Nat.7
    Nat.15 + Nat.2 = Nat.1.read(Nat.7)
    Nat.1.read(Nat.7) = Nat.17
    Nat.15 + Nat.2 = Nat.17
}

/// `5 * 3 + 2 = 17`.
theorem nat_five_three_add_two {
    Nat.5 * Nat.3 + Nat.2 = Nat.17
} by {
    nat_mul_5_3
    Nat.5 * Nat.3 = Nat.15
    Nat.5 * Nat.3 + Nat.2 = Nat.15 + Nat.2
    nat_add_15_2
    Nat.15 + Nat.2 = Nat.17
    Nat.5 * Nat.3 + Nat.2 = Nat.17
}

/// `17 + 8 = 25`.
theorem nat_add_17_8 {
    Nat.17 + Nat.8 = Nat.25
} by {
    Nat.17 = Nat.1.read(Nat.7)
    read_add_single(Nat.1, Nat.7, Nat.8)
    Nat.1.read(Nat.7) + Nat.8 = Nat.1.read(Nat.7 + Nat.8)
    nat_add_7_8_15
    Nat.7 + Nat.8 = Nat.15
    Nat.17 + Nat.8 = Nat.1.read(Nat.15)
    Nat.1.read(Nat.15) = Nat.25
    Nat.17 + Nat.8 = Nat.25
}

/// `1 < 11`.
theorem lt_one_eleven {
    Nat.1 < Nat.11
} by {
    lt_add_suc(Nat.1, Nat.9)
    Nat.1 < Nat.1 + Nat.10
    Nat.1 + Nat.10 = Nat.11
    Nat.1 < Nat.11
}

/// `1 < 17`.
theorem lt_one_seventeen {
    Nat.1 < Nat.17
} by {
    lt_one_eleven
    Nat.1 < Nat.11
    nat_add_11_6
    Nat.11 + Nat.6 = Nat.17
    exists(d: Nat) { Nat.11 + d = Nat.17 }
    Nat.11 <= Nat.17
    lt_ne(Nat.11, Nat.17, Nat.6)
    Nat.6 != Nat.0
    Nat.11 != Nat.17
    Nat.11 < Nat.17
    Nat.1 + Nat.10 = Nat.11
    exists(d: Nat) { Nat.1 + d = Nat.11 }
    Nat.1 <= Nat.11
    lte_and_lt(Nat.1, Nat.11, Nat.17)
    Nat.1 < Nat.17
}

/// `1 < 187`.
theorem lt_one_187 {
    Nat.1 < Nat.187
} by {
    read_pos(Nat.1, Nat.7)
    Nat.0 < Nat.1
    Nat.0 < Nat.17
    Nat.17 != Nat.0
    lt_one_eleven
    Nat.1 < Nat.11
    lt_mul_both(Nat.17, Nat.1, Nat.11)
    Nat.17 != Nat.0 and Nat.1 < Nat.11
    Nat.17 * Nat.1 < Nat.17 * Nat.11
    Nat.17 * Nat.1 = Nat.17
    mul_comm(Nat.17, Nat.11)
    Nat.17 * Nat.11 = Nat.11 * Nat.17
    nat_mul_11_17
    Nat.11 * Nat.17 = Nat.187
    Nat.17 * Nat.11 = Nat.187
    Nat.17 < Nat.187
    Nat.1 + Nat.16 = Nat.17
    exists(d: Nat) { Nat.1 + d = Nat.17 }
    Nat.1 <= Nat.17
    lte_and_lt(Nat.1, Nat.17, Nat.187)
    Nat.1 < Nat.187
}

/// `0 < 560`.
theorem lt_zero_560 {
    Nat.0 < Nat.560
} by {
    lt_suc(Nat.0)
    Nat.0 < Nat.1
    lt_imp_lt_suc(Nat.0, Nat.1)
    Nat.0 < Nat.2
    lt_imp_lt_suc(Nat.0, Nat.2)
    Nat.0 < Nat.3
    lt_imp_lt_suc(Nat.0, Nat.3)
    Nat.0 < Nat.4
    lt_imp_lt_suc(Nat.0, Nat.4)
    Nat.0 < Nat.5
    read_pos(Nat.5, Nat.6)
    Nat.0 < Nat.5.read(Nat.6)
    Nat.5.read(Nat.6) = Nat.56
    Nat.0 < Nat.56
    read_pos(Nat.56, Nat.0)
    Nat.0 < Nat.56.read(Nat.0)
    Nat.56.read(Nat.0) = Nat.560
    Nat.0 < Nat.560
}

/// `11 != 17`.
theorem eleven_ne_seventeen {
    Nat.11 != Nat.17
} by {
    nat_add_11_6
    Nat.11 + Nat.6 = Nat.17
    lt_ne(Nat.11, Nat.17, Nat.6)
    Nat.6 != Nat.0
    Nat.11 != Nat.17
}

/// `17 != 3`.
theorem seventeen_ne_three {
    Nat.17 != Nat.3
} by {
    nat_add_3_14
    Nat.3 + Nat.14 = Nat.17
    lt_ne(Nat.3, Nat.17, Nat.14)
    read_pos(Nat.1, Nat.4)
    Nat.0 < Nat.1
    Nat.0 < Nat.14
    Nat.14 != Nat.0
    Nat.17 != Nat.3
}

// ---------------------------------------------------------------------------
// Primality of 11 and 17.
//
// The composite-contradiction pattern: a composite `p = b * c` with `1 < b`
// and `1 < c` has a factor `b` with `b <= sqrt(p)`; checking the few small
// divisors is enough.
// ---------------------------------------------------------------------------

/// `2` does not divide `11`.
theorem not_two_divides_eleven {
    not Nat.2.divides(Nat.11)
} by {
    if Nat.2.divides(Nat.11) {
        div_imp_mod(Nat.11, Nat.2)
        Nat.11.mod(Nat.2) = Nat.0
        mod_add_mul(Nat.5, Nat.2, Nat.1)
        (Nat.5 * Nat.2 + Nat.1).mod(Nat.2) = Nat.1.mod(Nat.2)
        Nat.5 * Nat.2 + Nat.1 = Nat.11
        Nat.11.mod(Nat.2) = Nat.1.mod(Nat.2)
        lt_one_two
        Nat.1 < Nat.2
        small_mod(Nat.1, Nat.2)
        Nat.1.mod(Nat.2) = Nat.1
        Nat.11.mod(Nat.2) = Nat.1
        Nat.0 = Nat.1
        one_ne_zero
        false
    }
}

/// `3` does not divide `11`.
theorem not_three_divides_eleven {
    not Nat.3.divides(Nat.11)
} by {
    if Nat.3.divides(Nat.11) {
        div_imp_mod(Nat.11, Nat.3)
        Nat.11.mod(Nat.3) = Nat.0
        mod_add_mul(Nat.3, Nat.3, Nat.2)
        (Nat.3 * Nat.3 + Nat.2).mod(Nat.3) = Nat.2.mod(Nat.3)
        Nat.3 * Nat.3 + Nat.2 = Nat.11
        Nat.11.mod(Nat.3) = Nat.2.mod(Nat.3)
        lt_two_three
        Nat.2 < Nat.3
        small_mod(Nat.2, Nat.3)
        Nat.2.mod(Nat.3) = Nat.2
        Nat.11.mod(Nat.3) = Nat.2
        Nat.0 = Nat.2
        Nat.2 != Nat.0
        false
    }
}

/// No `b` with `1 < b` and `1 < c` can satisfy `11 = b * c`.
theorem eleven_composite_contradiction(b: Nat, c: Nat) {
    Nat.1 < b and Nat.1 < c implies not (Nat.11 = b * c)
} by {
    if Nat.1 < b and Nat.1 < c {
        if Nat.11 = b * c {
            Nat.11 = b * c
            exists(x: Nat) { b * x = Nat.11 }
            b.divides(Nat.11)
            if b <= Nat.3 {
                lt_suc(Nat.3)
                Nat.3 < Nat.4
                lte_and_lt(b, Nat.3, Nat.4)
                b < Nat.4
                lt_suc_right(b, Nat.3)
                if b = Nat.3 {
                    Nat.3.divides(Nat.11)
                    not_three_divides_eleven
                    false
                } else {
                    b < Nat.3
                    lt_suc_right(b, Nat.2)
                    if b = Nat.2 {
                        Nat.2.divides(Nat.11)
                        not_two_divides_eleven
                        false
                    } else {
                        b < Nat.2
                        lt_imp_lte_suc(Nat.1, b)
                        Nat.2 <= b
                        lte_and_lt(Nat.2, b, Nat.2)
                        Nat.2 < Nat.2
                        lt_not_ref(Nat.2)
                        false
                    }
                }
            } else {
                not (b <= Nat.3)
                lt_or_lte(Nat.3, b)
                Nat.3 < b or b <= Nat.3
                Nat.3 < b
                lt_imp_lte_suc(Nat.3, b)
                Nat.4 <= b
                if c <= Nat.3 {
                    Nat.11 = b * c
                    exists(x: Nat) { c * x = Nat.11 }
                    c.divides(Nat.11)
                    lt_suc(Nat.3)
                    Nat.3 < Nat.4
                    lte_and_lt(c, Nat.3, Nat.4)
                    c < Nat.4
                    lt_suc_right(c, Nat.3)
                    if c = Nat.3 {
                        Nat.3.divides(Nat.11)
                        not_three_divides_eleven
                        false
                    } else {
                        c < Nat.3
                        lt_suc_right(c, Nat.2)
                        if c = Nat.2 {
                            Nat.2.divides(Nat.11)
                            not_two_divides_eleven
                            false
                        } else {
                            c < Nat.2
                            lt_imp_lte_suc(Nat.1, c)
                            Nat.2 <= c
                            lte_and_lt(Nat.2, c, Nat.2)
                            Nat.2 < Nat.2
                            lt_not_ref(Nat.2)
                            false
                        }
                    }
                } else {
                    not (c <= Nat.3)
                    lt_or_lte(Nat.3, c)
                    Nat.3 < c or c <= Nat.3
                    Nat.3 < c
                    lt_imp_lte_suc(Nat.3, c)
                    Nat.4 <= c
                    lte_mul_both(Nat.4, Nat.4, c)
                    Nat.4 <= Nat.4 implies Nat.4 * Nat.4 <= Nat.4 * c
                    Nat.4 * Nat.4 <= Nat.4 * c
                    nat_mul_4_4
                    Nat.4 * Nat.4 = Nat.16
                    Nat.16 <= Nat.4 * c
                    lte_mul_both(c, Nat.4, b)
                    Nat.4 <= b implies c * Nat.4 <= c * b
                    c * Nat.4 <= c * b
                    mul_comm(c, Nat.4)
                    c * Nat.4 = Nat.4 * c
                    mul_comm(c, b)
                    c * b = b * c
                    Nat.4 * c <= b * c
                    lte_trans(Nat.16, Nat.4 * c, b * c)
                    Nat.16 <= b * c
                    Nat.11 = b * c
                    Nat.16 <= Nat.11
                    Nat.11 + Nat.5 = Nat.16
                    exists(d: Nat) { Nat.11 + d = Nat.16 }
                    Nat.11 <= Nat.16
                    lt_ne(Nat.11, Nat.16, Nat.5)
                    Nat.5 != Nat.0
                    Nat.11 != Nat.16
                    Nat.11 < Nat.16
                    lte_imp_not_lt(Nat.16, Nat.11)
                    not (Nat.11 < Nat.16)
                    false
                }
            }
        }
        not (Nat.11 = b * c)
    }
}

/// Eleven is prime.
theorem eleven_is_prime {
    Nat.11.is_prime
} by {
    lt_one_eleven
    Nat.1 < Nat.11
    if Nat.11.is_composite {
        Nat.11.is_composite = exists(b: Nat, c: Nat) {
            Nat.1 < b and Nat.1 < c and Nat.11 = b * c
        }
        let (b: Nat, c: Nat) satisfy {
            Nat.1 < b and Nat.1 < c and Nat.11 = b * c
        }
        eleven_composite_contradiction(b, c)
        Nat.1 < b and Nat.1 < c implies not (Nat.11 = b * c)
        not (Nat.11 = b * c)
        Nat.11 = b * c
        false
    }
    not Nat.11.is_composite
    Nat.11.is_prime = (Nat.1 < Nat.11 and not Nat.11.is_composite)
    Nat.11.is_prime
}

/// `8 * 2 + 1 = 17`.
theorem nat_eight_two_add_one {
    Nat.8 * Nat.2 + Nat.1 = Nat.17
} by {
    nat_mul_8_2
    Nat.8 * Nat.2 = Nat.16
    Nat.8 * Nat.2 + Nat.1 = Nat.16 + Nat.1
    Nat.16 + Nat.1 = Nat.17
    Nat.8 * Nat.2 + Nat.1 = Nat.17
}

/// `4 * 4 + 1 = 17`.
theorem nat_four_four_add_one {
    Nat.4 * Nat.4 + Nat.1 = Nat.17
} by {
    nat_mul_4_4
    Nat.4 * Nat.4 = Nat.16
    Nat.4 * Nat.4 + Nat.1 = Nat.16 + Nat.1
    Nat.16 + Nat.1 = Nat.17
    Nat.4 * Nat.4 + Nat.1 = Nat.17
}

/// `2` does not divide `17`.
theorem not_two_divides_seventeen {
    not Nat.2.divides(Nat.17)
} by {
    if Nat.2.divides(Nat.17) {
        div_imp_mod(Nat.17, Nat.2)
        Nat.17.mod(Nat.2) = Nat.0
        mod_add_mul(Nat.8, Nat.2, Nat.1)
        (Nat.8 * Nat.2 + Nat.1).mod(Nat.2) = Nat.1.mod(Nat.2)
        nat_eight_two_add_one
        Nat.8 * Nat.2 + Nat.1 = Nat.17
        Nat.17.mod(Nat.2) = Nat.1.mod(Nat.2)
        lt_one_two
        Nat.1 < Nat.2
        small_mod(Nat.1, Nat.2)
        Nat.1.mod(Nat.2) = Nat.1
        Nat.17.mod(Nat.2) = Nat.1
        Nat.0 = Nat.1
        one_ne_zero
        false
    }
}

/// `3` does not divide `17`.
theorem not_three_divides_seventeen {
    not Nat.3.divides(Nat.17)
} by {
    if Nat.3.divides(Nat.17) {
        div_imp_mod(Nat.17, Nat.3)
        Nat.17.mod(Nat.3) = Nat.0
        mod_add_mul(Nat.5, Nat.3, Nat.2)
        (Nat.5 * Nat.3 + Nat.2).mod(Nat.3) = Nat.2.mod(Nat.3)
        nat_five_three_add_two
        Nat.5 * Nat.3 + Nat.2 = Nat.17
        Nat.17.mod(Nat.3) = Nat.2.mod(Nat.3)
        lt_two_three
        Nat.2 < Nat.3
        small_mod(Nat.2, Nat.3)
        Nat.2.mod(Nat.3) = Nat.2
        Nat.17.mod(Nat.3) = Nat.2
        Nat.0 = Nat.2
        Nat.2 != Nat.0
        false
    }
}

/// `4` does not divide `17`.
theorem not_four_divides_seventeen {
    not Nat.4.divides(Nat.17)
} by {
    if Nat.4.divides(Nat.17) {
        div_imp_mod(Nat.17, Nat.4)
        Nat.17.mod(Nat.4) = Nat.0
        mod_add_mul(Nat.4, Nat.4, Nat.1)
        (Nat.4 * Nat.4 + Nat.1).mod(Nat.4) = Nat.1.mod(Nat.4)
        nat_four_four_add_one
        Nat.4 * Nat.4 + Nat.1 = Nat.17
        Nat.17.mod(Nat.4) = Nat.1.mod(Nat.4)
        lt_one_four
        Nat.1 < Nat.4
        small_mod(Nat.1, Nat.4)
        Nat.1.mod(Nat.4) = Nat.1
        Nat.17.mod(Nat.4) = Nat.1
        Nat.0 = Nat.1
        one_ne_zero
        false
    }
}

/// No `b` with `1 < b` and `1 < c` can satisfy `17 = b * c`.
theorem seventeen_composite_contradiction(b: Nat, c: Nat) {
    Nat.1 < b and Nat.1 < c implies not (Nat.17 = b * c)
} by {
    if Nat.1 < b and Nat.1 < c {
        if Nat.17 = b * c {
            Nat.17 = b * c
            exists(x: Nat) { b * x = Nat.17 }
            b.divides(Nat.17)
            if b <= Nat.4 {
                lt_suc(Nat.4)
                Nat.4 < Nat.5
                lte_and_lt(b, Nat.4, Nat.5)
                b < Nat.5
                lt_suc_right(b, Nat.4)
                if b = Nat.4 {
                    Nat.4.divides(Nat.17)
                    not_four_divides_seventeen
                    false
                } else {
                    b < Nat.4
                    lt_suc_right(b, Nat.3)
                    if b = Nat.3 {
                        Nat.3.divides(Nat.17)
                        not_three_divides_seventeen
                        false
                    } else {
                        b < Nat.3
                        lt_suc_right(b, Nat.2)
                        if b = Nat.2 {
                            Nat.2.divides(Nat.17)
                            not_two_divides_seventeen
                            false
                        } else {
                            b < Nat.2
                            lt_imp_lte_suc(Nat.1, b)
                            Nat.2 <= b
                            lte_and_lt(Nat.2, b, Nat.2)
                            Nat.2 < Nat.2
                            lt_not_ref(Nat.2)
                            false
                        }
                    }
                }
            } else {
                not (b <= Nat.4)
                lt_or_lte(Nat.4, b)
                Nat.4 < b or b <= Nat.4
                Nat.4 < b
                lt_imp_lte_suc(Nat.4, b)
                Nat.5 <= b
                if c <= Nat.4 {
                    Nat.17 = b * c
                    exists(x: Nat) { c * x = Nat.17 }
                    c.divides(Nat.17)
                    lt_suc(Nat.4)
                    Nat.4 < Nat.5
                    lte_and_lt(c, Nat.4, Nat.5)
                    c < Nat.5
                    lt_suc_right(c, Nat.4)
                    if c = Nat.4 {
                        Nat.4.divides(Nat.17)
                        not_four_divides_seventeen
                        false
                    } else {
                        c < Nat.4
                        lt_suc_right(c, Nat.3)
                        if c = Nat.3 {
                            Nat.3.divides(Nat.17)
                            not_three_divides_seventeen
                            false
                        } else {
                            c < Nat.4
                            lt_suc_right(c, Nat.2)
                            if c = Nat.2 {
                                Nat.2.divides(Nat.17)
                                not_two_divides_seventeen
                                false
                            } else {
                                c < Nat.2
                                lt_imp_lte_suc(Nat.1, c)
                                Nat.2 <= c
                                lte_and_lt(Nat.2, c, Nat.2)
                                Nat.2 < Nat.2
                                lt_not_ref(Nat.2)
                                false
                            }
                        }
                    }
                } else {
                    not (c <= Nat.4)
                    lt_or_lte(Nat.4, c)
                    Nat.4 < c or c <= Nat.4
                    Nat.4 < c
                    lt_imp_lte_suc(Nat.4, c)
                    Nat.5 <= c
                    lte_mul_both(Nat.5, Nat.5, c)
                    Nat.5 <= Nat.5 implies Nat.5 * Nat.5 <= Nat.5 * c
                    Nat.5 * Nat.5 <= Nat.5 * c
                    nat_mul_5_5
                    Nat.5 * Nat.5 = Nat.25
                    Nat.25 <= Nat.5 * c
                    lte_mul_both(c, Nat.5, b)
                    Nat.5 <= b implies c * Nat.5 <= c * b
                    c * Nat.5 <= c * b
                    mul_comm(c, Nat.5)
                    c * Nat.5 = Nat.5 * c
                    mul_comm(c, b)
                    c * b = b * c
                    Nat.5 * c <= b * c
                    lte_trans(Nat.25, Nat.5 * c, b * c)
                    Nat.25 <= b * c
                    Nat.17 = b * c
                    Nat.25 <= Nat.17
                    nat_add_17_8
                    Nat.17 + Nat.8 = Nat.25
                    exists(d: Nat) { Nat.17 + d = Nat.25 }
                    Nat.17 <= Nat.25
                    lt_ne(Nat.17, Nat.25, Nat.8)
                    Nat.8 != Nat.0
                    Nat.17 != Nat.25
                    Nat.17 < Nat.25
                    lte_imp_not_lt(Nat.25, Nat.17)
                    not (Nat.17 < Nat.25)
                    false
                }
            }
        }
        not (Nat.17 = b * c)
    }
}

/// Seventeen is prime.
theorem seventeen_is_prime {
    Nat.17.is_prime
} by {
    lt_one_seventeen
    Nat.1 < Nat.17
    if Nat.17.is_composite {
        Nat.17.is_composite = exists(b: Nat, c: Nat) {
            Nat.1 < b and Nat.1 < c and Nat.17 = b * c
        }
        let (b: Nat, c: Nat) satisfy {
            Nat.1 < b and Nat.1 < c and Nat.17 = b * c
        }
        seventeen_composite_contradiction(b, c)
        Nat.1 < b and Nat.1 < c implies not (Nat.17 = b * c)
        not (Nat.17 = b * c)
        Nat.17 = b * c
        false
    }
    not Nat.17.is_composite
    Nat.17.is_prime = (Nat.1 < Nat.17 and not Nat.17.is_composite)
    Nat.17.is_prime
}

// ---------------------------------------------------------------------------
// Pairwise coprimality of 3, 11 and 17.
// ---------------------------------------------------------------------------

/// `3` and `11` are coprime.
theorem three_coprime_eleven {
    Nat.3.coprime(Nat.11)
} by {
    coprime_of_distinct_primes(Nat.3, Nat.11)
    three_is_prime
    Nat.3.is_prime
    eleven_is_prime
    Nat.11.is_prime
    Nat.3 != Nat.11
    Nat.3.coprime(Nat.11)
}

/// `17` and `3` are coprime.
theorem seventeen_coprime_three {
    Nat.17.coprime(Nat.3)
} by {
    coprime_of_distinct_primes(Nat.17, Nat.3)
    seventeen_is_prime
    Nat.17.is_prime
    three_is_prime
    Nat.3.is_prime
    seventeen_ne_three
    Nat.17 != Nat.3
    Nat.17.coprime(Nat.3)
}

/// `17` and `11` are coprime.
theorem seventeen_coprime_eleven {
    Nat.17.coprime(Nat.11)
} by {
    coprime_of_distinct_primes(Nat.17, Nat.11)
    seventeen_is_prime
    Nat.17.is_prime
    eleven_is_prime
    Nat.11.is_prime
    eleven_ne_seventeen
    Nat.11 != Nat.17
    Nat.17.coprime(Nat.11)
}

/// `3 * 11` and `17` are coprime.
theorem thirty_three_coprime_seventeen {
    (Nat.3 * Nat.11).coprime(Nat.17)
} by {
    coprime_mul(Nat.17, Nat.3, Nat.11)
    seventeen_coprime_three
    Nat.17.coprime(Nat.3)
    seventeen_coprime_eleven
    Nat.17.coprime(Nat.11)
    Nat.17.coprime(Nat.3 * Nat.11)
    coprime_comm(Nat.17, Nat.3 * Nat.11)
    (Nat.3 * Nat.11).coprime(Nat.17)
}

// ---------------------------------------------------------------------------
// 561 is a Carmichael number.
// ---------------------------------------------------------------------------

/// A unit modulo `561` is a unit modulo each prime factor.
theorem coprime_561_imp_coprime_factors(a: Nat) {
    a.coprime(Nat.561) implies (a.coprime(Nat.3) and a.coprime(Nat.11)
        and a.coprime(Nat.17))
} by {
    if a.coprime(Nat.561) {
        nat_three_eleven_seventeen
        Nat.3 * Nat.11 * Nat.17 = Nat.561
        a.coprime(Nat.3 * Nat.11 * Nat.17)
        coprime_mul_iff(a, Nat.3 * Nat.11, Nat.17)
        a.coprime((Nat.3 * Nat.11) * Nat.17) implies (a.coprime(Nat.3 * Nat.11) and a.coprime(Nat.17))
        a.coprime(Nat.3 * Nat.11) and a.coprime(Nat.17)
        a.coprime(Nat.3 * Nat.11)
        coprime_mul_iff(a, Nat.3, Nat.11)
        a.coprime(Nat.3 * Nat.11) implies (a.coprime(Nat.3) and a.coprime(Nat.11))
        a.coprime(Nat.3) and a.coprime(Nat.11)
        a.coprime(Nat.3)
        a.coprime(Nat.11)
        a.coprime(Nat.17)
        a.coprime(Nat.3) and a.coprime(Nat.11) and a.coprime(Nat.17)
    }
}

/// For a unit `a` modulo `3`, `a^560 ≡ 1 (mod 3)`: Fermat at the exponent
/// `2` lifted along `560 = 2 * 280`.
theorem a_pow_560_congr_one_mod_three(a: Nat) {
    a.coprime(Nat.3) implies a.pow(Nat.560).congr_mod(Nat.1, Nat.3)
} by {
    if a.coprime(Nat.3) {
        euler(Nat.3, a)
        Nat.3 != Nat.0
        a.coprime(Nat.3)
        a.pow(Nat.3.totient).congr_mod(Nat.1, Nat.3)
        totient_prime(Nat.3)
        three_is_prime
        Nat.3.is_prime
        Nat.3.totient = Nat.3 - Nat.1
        Nat.3 - Nat.1 = Nat.2
        Nat.3.totient = Nat.2
        a.pow(Nat.2).congr_mod(Nat.1, Nat.3)
        congr_mod_pow(a.pow(Nat.2), Nat.1, Nat.3, Nat.280)
        a.pow(Nat.2).pow(Nat.280).congr_mod(Nat.1.pow(Nat.280), Nat.3)
        exp_mul(a, Nat.2, Nat.280)
        a.pow(Nat.2 * Nat.280) = a.pow(Nat.2).pow(Nat.280)
        nat_mul_2_280
        Nat.2 * Nat.280 = Nat.560
        a.pow(Nat.560) = a.pow(Nat.2).pow(Nat.280)
        a.pow(Nat.560).congr_mod(Nat.1.pow(Nat.280), Nat.3)
        one_exp(Nat.280)
        Nat.1.pow(Nat.280) = Nat.1
        a.pow(Nat.560).congr_mod(Nat.1, Nat.3)
    }
}

/// For a unit `a` modulo `11`, `a^560 ≡ 1 (mod 11)`: Fermat at the exponent
/// `10` lifted along `560 = 10 * 56`.
theorem a_pow_560_congr_one_mod_eleven(a: Nat) {
    a.coprime(Nat.11) implies a.pow(Nat.560).congr_mod(Nat.1, Nat.11)
} by {
    if a.coprime(Nat.11) {
        euler(Nat.11, a)
        Nat.11 != Nat.0
        a.coprime(Nat.11)
        a.pow(Nat.11.totient).congr_mod(Nat.1, Nat.11)
        totient_prime(Nat.11)
        eleven_is_prime
        Nat.11.is_prime
        Nat.11.totient = Nat.11 - Nat.1
        Nat.11 - Nat.1 = Nat.10
        Nat.11.totient = Nat.10
        a.pow(Nat.10).congr_mod(Nat.1, Nat.11)
        congr_mod_pow(a.pow(Nat.10), Nat.1, Nat.11, Nat.56)
        a.pow(Nat.10).pow(Nat.56).congr_mod(Nat.1.pow(Nat.56), Nat.11)
        exp_mul(a, Nat.10, Nat.56)
        a.pow(Nat.10 * Nat.56) = a.pow(Nat.10).pow(Nat.56)
        Nat.10 * Nat.56 = Nat.560
        a.pow(Nat.560) = a.pow(Nat.10).pow(Nat.56)
        a.pow(Nat.560).congr_mod(Nat.1.pow(Nat.56), Nat.11)
        one_exp(Nat.56)
        Nat.1.pow(Nat.56) = Nat.1
        a.pow(Nat.560).congr_mod(Nat.1, Nat.11)
    }
}

/// For a unit `a` modulo `17`, `a^560 ≡ 1 (mod 17)`: Fermat at the exponent
/// `16` lifted along `560 = 16 * 35`.
theorem a_pow_560_congr_one_mod_seventeen(a: Nat) {
    a.coprime(Nat.17) implies a.pow(Nat.560).congr_mod(Nat.1, Nat.17)
} by {
    if a.coprime(Nat.17) {
        euler(Nat.17, a)
        Nat.17 != Nat.0
        a.coprime(Nat.17)
        a.pow(Nat.17.totient).congr_mod(Nat.1, Nat.17)
        totient_prime(Nat.17)
        seventeen_is_prime
        Nat.17.is_prime
        Nat.17.totient = Nat.17 - Nat.1
        nat_sub_17_1
        Nat.17 - Nat.1 = Nat.16
        Nat.17.totient = Nat.16
        a.pow(Nat.16).congr_mod(Nat.1, Nat.17)
        congr_mod_pow(a.pow(Nat.16), Nat.1, Nat.17, Nat.35)
        a.pow(Nat.16).pow(Nat.35).congr_mod(Nat.1.pow(Nat.35), Nat.17)
        exp_mul(a, Nat.16, Nat.35)
        a.pow(Nat.16 * Nat.35) = a.pow(Nat.16).pow(Nat.35)
        nat_mul_16_35
        Nat.16 * Nat.35 = Nat.560
        a.pow(Nat.560) = a.pow(Nat.16).pow(Nat.35)
        a.pow(Nat.560).congr_mod(Nat.1.pow(Nat.35), Nat.17)
        one_exp(Nat.35)
        Nat.1.pow(Nat.35) = Nat.1
        a.pow(Nat.560).congr_mod(Nat.1, Nat.17)
    }
}

/// `560` is a universal exponent modulo `561`: every unit `a` satisfies
/// `a^560 ≡ 1 (mod 561)`.
theorem carmichael_witness_561 {
    carmichael_witness(Nat.561)(Nat.560)
} by {
    carmichael_witness_apply(Nat.561, Nat.560)
    carmichael_witness(Nat.561)(Nat.560) =
        (Nat.0 < Nat.560 and forall(a: Nat) {
            a.coprime(Nat.561) implies a.pow(Nat.560).congr_mod(Nat.1, Nat.561)
        })
    lt_zero_560
    Nat.0 < Nat.560
    forall(a: Nat) {
        if a.coprime(Nat.561) {
            coprime_561_imp_coprime_factors(a)
            a.coprime(Nat.3) and a.coprime(Nat.11) and a.coprime(Nat.17)
            a.coprime(Nat.3)
            a.coprime(Nat.11)
            a.coprime(Nat.17)
            a_pow_560_congr_one_mod_three(a)
            a.pow(Nat.560).congr_mod(Nat.1, Nat.3)
            a_pow_560_congr_one_mod_eleven(a)
            a.pow(Nat.560).congr_mod(Nat.1, Nat.11)
            a_pow_560_congr_one_mod_seventeen(a)
            a.pow(Nat.560).congr_mod(Nat.1, Nat.17)
            nat_congr_combine_coprime(Nat.3, Nat.11, a.pow(Nat.560), Nat.1)
            three_coprime_eleven
            Nat.3.coprime(Nat.11)
            a.pow(Nat.560).congr_mod(Nat.1, Nat.3 * Nat.11)
            nat_congr_combine_coprime(Nat.3 * Nat.11, Nat.17,
                a.pow(Nat.560), Nat.1)
            thirty_three_coprime_seventeen
            (Nat.3 * Nat.11).coprime(Nat.17)
            a.pow(Nat.560).congr_mod(Nat.1, (Nat.3 * Nat.11) * Nat.17)
            nat_three_eleven_seventeen
            Nat.3 * Nat.11 * Nat.17 = Nat.561
            a.pow(Nat.560).congr_mod(Nat.1, Nat.561)
        }
        a.coprime(Nat.561) implies a.pow(Nat.560).congr_mod(Nat.1, Nat.561)
    }
    Nat.0 < Nat.560 and forall(a: Nat) {
        a.coprime(Nat.561) implies a.pow(Nat.560).congr_mod(Nat.1, Nat.561)
    }
    carmichael_witness(Nat.561)(Nat.560)
}

/// A Carmichael number: composite `n` with `a^(n-1) ≡ 1 (mod n)` for every
/// unit `a` modulo `n`.
define is_carmichael_number(n: Nat) -> Bool {
    n.is_composite and carmichael_witness(n)(n - Nat.1)
}

/// The defining predicate unfolds to compositeness plus the universal
/// congruence at `n - 1`.
theorem is_carmichael_number_apply(n: Nat) {
    is_carmichael_number(n) =
        (n.is_composite and carmichael_witness(n)(n - Nat.1))
}

/// `561` is composite: `561 = 3 * 187` with both factors above one.
theorem composite_561 {
    Nat.561.is_composite
} by {
    Nat.1 < Nat.3
    lt_one_187
    Nat.1 < Nat.187
    nat_mul_3_187
    Nat.3 * Nat.187 = Nat.561
    exists(c: Nat) { Nat.1 < Nat.3 and Nat.1 < c and Nat.561 = Nat.3 * c }
    exists(b: Nat, c: Nat) {
        Nat.1 < b and Nat.1 < c and Nat.561 = b * c
    }
    Nat.561.is_composite = exists(b: Nat, c: Nat) {
        Nat.1 < b and Nat.1 < c and Nat.561 = b * c
    }
    Nat.561.is_composite
}

/// `561` is a Carmichael number.
theorem is_carmichael_number_561 {
    is_carmichael_number(Nat.561)
} by {
    composite_561
    Nat.561.is_composite
    carmichael_witness_561
    carmichael_witness(Nat.561)(Nat.560)
    Nat.561 - Nat.1 = Nat.560
    carmichael_witness(Nat.561)(Nat.561 - Nat.1)
    Nat.561.is_composite and carmichael_witness(Nat.561)(Nat.561 - Nat.1)
    is_carmichael_number_apply(Nat.561)
    is_carmichael_number(Nat.561) =
        (Nat.561.is_composite and carmichael_witness(Nat.561)(Nat.561 - Nat.1))
    is_carmichael_number(Nat.561)
}

// ---------------------------------------------------------------------------
// Korselt conditions for 561: 2 | 560, 10 | 560 and 16 | 560 (and 561 is
// squarefree with the prime factors 3, 11, 17).
// ---------------------------------------------------------------------------

/// `2 | 560`.
theorem two_divides_560 {
    Nat.2.divides(Nat.560)
} by {
    nat_mul_2_280
    Nat.2 * Nat.280 = Nat.560
    exists(c: Nat) { Nat.2 * c = Nat.560 }
    Nat.2.divides(Nat.560)
}

/// `10 | 560`.
theorem ten_divides_560 {
    Nat.10.divides(Nat.560)
} by {
    Nat.10 * Nat.56 = Nat.560
    exists(c: Nat) { Nat.10 * c = Nat.560 }
    Nat.10.divides(Nat.560)
}

/// `16 | 560`.
theorem sixteen_divides_560 {
    Nat.16.divides(Nat.560)
} by {
    nat_mul_16_35
    Nat.16 * Nat.35 = Nat.560
    exists(c: Nat) { Nat.16 * c = Nat.560 }
    Nat.16.divides(Nat.560)
}

// ---------------------------------------------------------------------------
// The Carmichael function divides Euler's totient, and the defining property.
// ---------------------------------------------------------------------------

/// λ(n) divides φ(n) for every positive modulus (restated from
/// `carmichael.ac`).
theorem carmichael_lambda_divides_totient(n: Nat) {
    n != Nat.0 implies carmichael(n).divides(n.totient)
} by {
    carmichael_divides_totient(n)
    n != Nat.0 implies carmichael(n).divides(n.totient)
}

/// For a prime power `p^k`, λ(p^k) divides φ(p^k).
theorem carmichael_divides_prime_power_totient(p: Nat, k: Nat) {
    p.is_prime implies carmichael(p.pow(k)).divides((p.pow(k)).totient)
} by {
    if p.is_prime {
        Nat.1 < p
        p != Nat.0
        exp_ne_zero(p, k)
        p.pow(k) != Nat.0
        carmichael_divides_totient(p.pow(k))
        carmichael(p.pow(k)).divides((p.pow(k)).totient)
    }
}

/// The defining property of λ: `a^λ(n) ≡ 1 (mod n)` for every unit `a`
/// (restated from `carmichael.ac`).
theorem carmichael_defining_pow_congr_one(n: Nat, a: Nat) {
    n != Nat.0 and a.coprime(n)
        implies a.pow(carmichael(n)).congr_mod(Nat.1, n)
} by {
    carmichael_pow_congr_one(n, a)
    n != Nat.0 and a.coprime(n) implies a.pow(carmichael(n)).congr_mod(Nat.1, n)
}

// ---------------------------------------------------------------------------
// Statements left for future work (not yet provable with the current library).
// ---------------------------------------------------------------------------

// Korselt's criterion: a composite `n` is a Carmichael number iff `n` is
// squarefree and `p - 1` divides `n - 1` for every prime `p` dividing `n`.
//
// The forward direction (the Korselt conditions imply the universal
// congruence) needs, in general form: the prime factorisation list of `n`
// (available in `factorisation.ac`), the pairwise coprimality of its entries
// (from squarefreeness), Fermat's theorem at each prime factor, and a
// list-level CRT combine over the factor list (`crt_list.ac` provides
// `satisfies_all_unique_mod_system_modulus` for pairwise-coprime positive
// moduli).  The concrete case `n = 561` is proved above
// (`carmichael_witness_561`).
//
// The converse direction additionally needs a primitive root modulo each
// prime `p | n` to force `p - 1 | n - 1` and a unit construction to rule out
// square divisors, both of which are not yet in the library.
//
// theorem korselt_criterion(n: Nat) {
//     (n.is_composite and carmichael_witness(n)(n - Nat.1)) =
//         (is_squarefree(n) and forall(p: Nat) {
//             p.is_prime and p.divides(n) implies (p - Nat.1).divides(n - Nat.1)
//         })
// }

// λ(p^k) = φ(p^k) = p^k - p^(k-1) for odd prime `p`.  The divisibility
// λ(p^k) | φ(p^k) is proved above (`carmichael_divides_prime_power_totient`);
// the reverse inequality needs an element of multiplicative order φ(p^k)
// modulo `p^k` (a primitive root), which the library does not yet provide
// (see the note at the end of `carmichael.ac`).
//
// theorem carmichael_prime_power_unconditional(p: Nat, k: Nat) {
//     p.is_prime and p != Nat.2 and Nat.1 <= k
//         implies carmichael(p.pow(k)) = (p.pow(k)).totient
// }

// λ(2^k) = 2^(k-2) for k >= 3.  Also not yet provable: it needs the
// order-`2^(k-2)` element `5` (or `3`) modulo `2^k` and the minimality
// machinery.
//
// theorem carmichael_two_power(k: Nat) {
//     Nat.3 <= k implies carmichael(Nat.2.pow(k)) = Nat.2.pow(k - Nat.2)
// }

// 561 is the smallest Carmichael number.  The minimality over all `n < 561`
// is an exhaustive check that is not formalised here; note also that the
// least universal exponent of 561 is λ(561) = lcm(λ(3), λ(11), λ(17)) =
// lcm(2, 10, 16) = 80, which divides 560, so `560` is a universal exponent
// but not the least one.
