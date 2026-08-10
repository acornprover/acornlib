/// Sums of squares: which natural numbers are sums of two squares, and the
/// small concrete cases.
///
/// The forward half of Fermat's two-squares theorem — a prime congruent to
/// one modulo four is a sum of two squares — is proved in
/// `number_theory/sum_of_two_squares.ac`, together with its converse.  This
/// file records the concrete representations (5, 13, 25), the failure for
/// three, the descent step behind the full characterisation (a prime
/// congruent to three modulo four divides a sum of two squares to an even
/// power), and the Brahmagupta-Fibonacci product identity over the reals.
/// The full characterisation (every prime congruent to three modulo four
/// divides `n` to an even power) and Lagrange's four-square theorem are
/// deep results recorded here as statements.
from nat import Nat, small_mod, read_add_single
from number_theory.sum_of_two_squares import prime_sum_of_two_squares,
    prime_sum_two_squares_converse
from number_theory.infinite_descent import three_mod_four, two_squares_descent
from number_theory.goldbach import three_is_prime, five_is_prime
from number_theory.primitive_root_applications2 import thirteen_is_prime
from number_theory.four_squares import is_sum_four_squares
from number_theory.congruence import mod_add_mul
from real import Real

numerals Nat
numerals Real

// ============================================================================
// Section 1: the predicate "sum of two squares"
// ============================================================================

/// A natural number is a sum of two squares when it has two witnesses.
define is_sum_two_squares(n: Nat) -> Bool {
    exists(a: Nat, b: Nat) { n = a * a + b * b }
}

/// Two explicit witnesses package an `is_sum_two_squares` proof.
theorem is_sum_two_squares_intro(n: Nat, a: Nat, b: Nat) {
    n = a * a + b * b implies is_sum_two_squares(n)
} by {
    if n = a * a + b * b {
        is_sum_two_squares(n) = exists(x: Nat, y: Nat) { n = x * x + y * y }
        exists(x: Nat, y: Nat) { n = x * x + y * y }
    }
}

/// A sum-of-two-squares proof supplies two witnesses.
theorem is_sum_two_squares_apply(n: Nat) {
    is_sum_two_squares(n) implies exists(a: Nat, b: Nat) { n = a * a + b * b }
} by {
    if is_sum_two_squares(n) {
        is_sum_two_squares(n) = exists(x: Nat, y: Nat) { n = x * x + y * y }
        exists(x: Nat, y: Nat) { n = x * x + y * y }
    }
}

// ============================================================================
// Section 2: the concrete representations 5, 13 and 25
// ============================================================================

/// Six plus nine is fifteen.  The numeral arithmetic is reduced to the
/// small additions the prover evaluates directly.
theorem six_add_nine {
    Nat.6 + Nat.9 = Nat.15
} by {
    Nat.6 + Nat.4 = Nat.10
    Nat.6 + Nat.9 = Nat.6 + (Nat.4 + Nat.5)
    Nat.6 + (Nat.4 + Nat.5) = (Nat.6 + Nat.4) + Nat.5
    Nat.10 + Nat.5 = Nat.15
    Nat.6 + Nat.9 = Nat.15
}

/// Five is a sum of two squares: 5 = 1² + 2².
theorem five_is_sum_two_squares {
    exists(a: Nat, b: Nat) { a * a + b * b = Nat.5 }
} by {
    Nat.1 * Nat.1 + Nat.2 * Nat.2 = Nat.5
    exists(a: Nat, b: Nat) { a * a + b * b = Nat.5 }
}

/// Thirteen is a sum of two squares: 13 = 2² + 3².
theorem thirteen_is_sum_two_squares {
    exists(a: Nat, b: Nat) { a * a + b * b = Nat.13 }
} by {
    Nat.2 * Nat.2 + Nat.3 * Nat.3 = Nat.13
    exists(a: Nat, b: Nat) { a * a + b * b = Nat.13 }
}

/// Twenty-five is a sum of two squares: 25 = 3² + 4².
theorem twenty_five_is_sum_two_squares {
    exists(a: Nat, b: Nat) { a * a + b * b = Nat.25 }
} by {
    six_add_nine
    Nat.6 + Nat.9 = Nat.15
    Nat.16 = Nat.1.read(Nat.6)
    read_add_single(Nat.1, Nat.6, Nat.9)
    Nat.1.read(Nat.6) + Nat.9 = Nat.1.read(Nat.6 + Nat.9)
    Nat.16 + Nat.9 = Nat.1.read(Nat.15)
    Nat.1.read(Nat.15) = Nat.25
    Nat.16 + Nat.9 = Nat.25
    Nat.3 * Nat.3 = Nat.9
    Nat.4 * Nat.4 = Nat.16
    Nat.3 * Nat.3 + Nat.4 * Nat.4 = Nat.16 + Nat.9
    Nat.3 * Nat.3 + Nat.4 * Nat.4 = Nat.25
    exists(a: Nat, b: Nat) { a * a + b * b = Nat.25 }
}

/// Twenty-five is also a sum of two squares through the square 5²: 25 = 0² + 5².
theorem twenty_five_is_sum_two_squares_zero_five {
    exists(a: Nat, b: Nat) { a * a + b * b = Nat.25 }
} by {
    Nat.0 * Nat.0 + Nat.5 * Nat.5 = Nat.25
    exists(a: Nat, b: Nat) { a * a + b * b = Nat.25 }
}

/// Five modulo four is one.
theorem five_mod_four {
    Nat.5.mod(Nat.4) = Nat.1
} by {
    mod_add_mul(Nat.1, Nat.4, Nat.1)
    (Nat.1 * Nat.4 + Nat.1).mod(Nat.4) = Nat.1.mod(Nat.4)
    Nat.1 * Nat.4 = Nat.4
    (Nat.4 + Nat.1).mod(Nat.4) = Nat.1.mod(Nat.4)
    small_mod(Nat.1, Nat.4)
    Nat.1.mod(Nat.4) = Nat.1
    Nat.5 = Nat.4 + Nat.1
    Nat.5.mod(Nat.4) = Nat.1
}

/// Thirteen modulo four is one.
theorem thirteen_mod_four {
    Nat.13.mod(Nat.4) = Nat.1
} by {
    mod_add_mul(Nat.3, Nat.4, Nat.1)
    (Nat.3 * Nat.4 + Nat.1).mod(Nat.4) = Nat.1.mod(Nat.4)
    Nat.3 * Nat.4 = Nat.12
    (Nat.12 + Nat.1).mod(Nat.4) = Nat.1.mod(Nat.4)
    small_mod(Nat.1, Nat.4)
    Nat.1.mod(Nat.4) = Nat.1
    Nat.13 = Nat.12 + Nat.1
    Nat.13.mod(Nat.4) = Nat.1
}

/// As a prime congruent to one modulo four, five is a sum of two squares:
/// this is `prime_sum_of_two_squares` applied to five.
theorem five_prime_sum_two_squares {
    exists(a: Nat, b: Nat) { a * a + b * b = Nat.5 }
} by {
    five_is_prime
    Nat.5.is_prime
    five_mod_four
    Nat.5.mod(Nat.4) = Nat.1
    prime_sum_of_two_squares(Nat.5)
    exists(a: Nat, b: Nat) { a * a + b * b = Nat.5 }
}

/// As a prime congruent to one modulo four, thirteen is a sum of two squares:
/// this is `prime_sum_of_two_squares` applied to thirteen.
theorem thirteen_prime_sum_two_squares {
    exists(a: Nat, b: Nat) { a * a + b * b = Nat.13 }
} by {
    thirteen_is_prime
    Nat.13.is_prime
    thirteen_mod_four
    Nat.13.mod(Nat.4) = Nat.1
    prime_sum_of_two_squares(Nat.13)
    exists(a: Nat, b: Nat) { a * a + b * b = Nat.13 }
}

// ============================================================================
// Section 3: three is not a sum of two squares
// ============================================================================

/// Three is not a sum of two squares.  A sum of two squares is congruent to
/// zero, one, or two modulo four (sum_of_two_squares.ac), and a prime that
/// is a sum of two squares and is not two is congruent to one modulo four;
/// three is a prime congruent to three modulo four.
theorem three_not_sum_two_squares {
    not exists(a: Nat, b: Nat) { a * a + b * b = Nat.3 }
} by {
    if exists(a: Nat, b: Nat) { a * a + b * b = Nat.3 } {
        three_is_prime
        Nat.3.is_prime
        Nat.3 != Nat.2
        prime_sum_two_squares_converse(Nat.3)
        Nat.3.mod(Nat.4) = Nat.1
        three_mod_four
        Nat.3.mod(Nat.4) = Nat.3
        Nat.3 != Nat.1
        false
    }
}

// ============================================================================
// Section 4: the descent step of the characterisation
// ============================================================================

/// A prime congruent to three modulo four that divides a sum of two squares
/// divides the sum by its square: the descent
/// (infinite_descent.ac) gives p | x and p | y, so x² + y² = p²(x'² + y'²)
/// with x = p·x', y = p·y'.  Iterating this step makes the exponent of p
/// even, which is the forward half of the characterisation below.
theorem three_mod_four_prime_square_divides(p: Nat, x: Nat, y: Nat) {
    p.is_prime and p.congr_mod(Nat.3, Nat.4) and p.divides(x * x + y * y)
        implies (p * p).divides(x * x + y * y)
} by {
    if p.is_prime and p.congr_mod(Nat.3, Nat.4) and p.divides(x * x + y * y) {
        two_squares_descent(p, x, y)
        p.divides(x) and p.divides(y)
        p.divides(x)
        p.divides(y)
        p.divides(x) = exists(c: Nat) { p * c = x }
        let c: Nat satisfy { p * c = x }
        p.divides(y) = exists(d: Nat) { p * d = y }
        let d: Nat satisfy { p * d = y }
        p * c = x
        p * d = y
        x * x = (p * c) * (p * c)
        (p * c) * (p * c) = (p * p) * (c * c)
        x * x = (p * p) * (c * c)
        y * y = (p * d) * (p * d)
        (p * d) * (p * d) = (p * p) * (d * d)
        y * y = (p * p) * (d * d)
        (p * p) * (c * c) + (p * p) * (d * d) = (p * p) * (c * c + d * d)
        x * x + y * y = (p * p) * (c * c) + (p * p) * (d * d)
        x * x + y * y = (p * p) * (c * c + d * d)
        exists(w: Nat) { (p * p) * w = x * x + y * y }
        (p * p).divides(x * x + y * y)
    }
}

// ============================================================================
// Section 5: the two-squares characterisation (statement)
// ============================================================================

/// A prime congruent to three modulo four divides `n` to an even power when
/// the exact exponent of the prime in `n` is even.
define three_mod_four_divides_to_even_power(p: Nat, n: Nat) -> Bool {
    (p.is_prime and p.mod(Nat.4) = Nat.3) implies
        exists(k: Nat) {
            p.pow(k).divides(n) and not p.pow(k + Nat.1).divides(n) and
            exists(j: Nat) { k = Nat.2 * j }
        }
}

/// The two-squares characterisation: a positive natural number is a sum of
/// two squares exactly when every prime congruent to three modulo four
/// divides it to an even power.
///
/// The forward direction iterates the descent step above: a prime p ≡ 3
/// (mod 4) dividing x² + y² divides both x and y, so p² divides the sum and
/// the reduced quotient is again a sum of two squares; strong induction on
/// the sum makes the exponent even.  The reverse direction factors n into
/// powers of two, of primes congruent to one modulo four (each a sum of two
/// squares by `prime_sum_of_two_squares`), and of primes congruent to three
/// modulo four to even powers (each such power is a sum of two squares by
/// repeated squaring), and combines the factors with the
/// Brahmagupta-Fibonacci identity below, which makes the sums of two squares
/// closed under multiplication.  Both directions are recorded here without
/// proof.
///
/// theorem two_squares_characterization(n: Nat) {
///     n != Nat.0 implies (is_sum_two_squares(n) = forall(p: Nat) {
///         three_mod_four_divides_to_even_power(p, n)
///     })
/// }

// ============================================================================
// Section 6: the Brahmagupta-Fibonacci product identity
// ============================================================================
//
// The identity (a² + b²)(c² + d²) = (ac - bd)² + (ad + bc)² shows that the
// sums of two squares are closed under multiplication.  It is proved here
// over the reals with the same expansion-and-cancellation lemmas used in
// theorems1000/theorem_brahmagupta_fibonacci.ac (which this file cannot
// import directly because the `top100` interface redefines `congr_mod`).
// The helpers are recorded first; each is a small, explicitly proved step.

/// A four-term sum with the last two terms moved to the front.
theorem real_add_four_last_next_to_first(a: Real, b: Real, c: Real, d: Real) {
    a + b + c + d = (a + d) + (b + c)
} by {
    a + b + c + d = ((a + b) + c) + d
    ((a + b) + c) + d = (a + b) + (c + d)
    (a + b) + (c + d) = a + (b + (c + d))
    b + (c + d) = (b + c) + d
    a + ((b + c) + d) = a + (d + (b + c))
    a + (d + (b + c)) = (a + d) + (b + c)
}

/// Reassociating two pairs: (a + b) + (c + d) = (a + c) + (b + d).
theorem real_add_pair_rearrange(a: Real, b: Real, c: Real, d: Real) {
    (a + b) + (c + d) = (a + c) + (b + d)
} by {
    (a + b) + (c + d) = a + b + c + d
    real_add_four_last_next_to_first(a, b, c, d)
    a + b + c + d = (a + d) + (b + c)
    (a + d) + (b + c) = a + (d + (b + c))
    a + (d + (b + c)) = a + (b + (d + c))
    d + c = c + d
    b + (d + c) = b + (c + d)
    a + (b + (d + c)) = a + (b + (c + d))
    b + (c + d) = (b + c) + d
    a + (b + (c + d)) = a + ((b + c) + d)
    (b + c) + d = (c + b) + d
    c + b = b + c
    a + ((b + c) + d) = a + ((c + b) + d)
    (c + b) + d = c + (b + d)
    a + ((c + b) + d) = a + (c + (b + d))
    a + (c + (b + d)) = (a + c) + (b + d)
}

/// The square of a difference expands to four terms.
theorem bf_square_sub(x: Real, y: Real) {
    (x - y) * (x - y) = x * x - x * y - y * x + y * y
} by {
    (x - y) * (x - y) = (x - y) * x - (x - y) * y
    (x - y) * x = x * x - y * x
    (x - y) * y = x * y - y * y
    (x - y) * (x - y) = x * x - y * x - (x * y - y * y)
    y * x = x * y
    x * x - y * x - (x * y - y * y) = x * x - x * y - (x * y - y * y)
    x * x - x * y - (x * y - y * y) = x * x - x * y + -(x * y - y * y)
    -(x * y - y * y) = -(x * y) + y * y
    x * x - x * y + -(x * y - y * y) = x * x - x * y + (-(x * y) + y * y)
    x * x - x * y + (-(x * y) + y * y) = x * x - x * y - x * y + y * y
    x * x - x * y - x * y + y * y = x * x - x * y - y * x + y * y
}

/// The square of a two-term sum expands to four terms.
theorem bf_square_add(x: Real, y: Real) {
    (x + y) * (x + y) = x * x + y * x + (x * y + y * y)
} by {
    (x + y) * (x + y) = (x + y) * x + (x + y) * y
    (x + y) * x = x * x + y * x
    (x + y) * y = x * y + y * y
    (x + y) * (x + y) = x * x + y * x + (x * y + y * y)
}

/// A difference of three terms is a sum of negations.
theorem bf_minus_to_add(x: Real, m: Real, n: Real, y: Real) {
    x - m - n + y = x + -m + -n + y
}

/// A sum of negations regroups as a difference of sums.
theorem bf_add_to_minus(x: Real, m: Real, n: Real, y: Real) {
    x + -m + -n + y = (x - m) + (y - n)
} by {
    x + -m + -n + y = (x + -m) + (-n + y)
    (x + -m) + (-n + y) = (x - m) + (y - n)
}

/// Cancelling a term and its negation: x - m + (m + z) = x + z.
theorem bf_cancel_sub_add(x: Real, m: Real, z: Real) {
    x - m + (m + z) = x + z
} by {
    x - m = x + -m
    x + -m + (m + z) = x + (-m + (m + z))
    -m + (m + z) = (-m + m) + z
    -m + m = Real.0
    (-m + m) + z = Real.0 + z
    Real.0 + z = z
    x + (-m + (m + z)) = x + z
}

/// Cancelling a swapped term and its negation: x - m + (z + m) = x + z.
theorem bf_cancel_swap_add(x: Real, m: Real, z: Real) {
    x - m + (z + m) = x + z
} by {
    z + m = m + z
    x - m + (z + m) = x - m + (m + z)
    bf_cancel_sub_add(x, m, z)
    x - m + (m + z) = x + z
}

/// Opposite middle terms cancel in a sum of two expanded squares:
/// (x - m - n + y) + (z + m + n + w) = x + y + z + w.
theorem bf_cancel_middle(x: Real, y: Real, z: Real, w: Real, m: Real, n: Real) {
    (x - m - n + y) + (z + m + n + w) = x + y + z + w
} by {
    bf_minus_to_add(x, m, n, y)
    x - m - n + y = x + -m + -n + y
    bf_add_to_minus(x, m, n, y)
    x + -m + -n + y = (x - m) + (y - n)
    z + m + n + w = (z + m) + (n + w)
    real_add_pair_rearrange(x - m, y - n, z + m, n + w)
    ((x - m) + (y - n)) + ((z + m) + (n + w)) =
        ((x - m) + (z + m)) + ((y - n) + (n + w))
    bf_cancel_swap_add(x, m, z)
    x - m + (z + m) = x + z
    bf_cancel_sub_add(y, n, w)
    y - n + (n + w) = y + w
    ((x - m) + (z + m)) + ((y - n) + (n + w)) = (x + z) + (y + w)
}

/// The four pure terms of the two expanded squares reassemble into the
/// expansion of the product.
theorem bf_pure_reassemble(a: Real, b: Real, c: Real, d: Real) {
    a * c * a * c + b * d * b * d + a * d * a * d + b * c * b * c =
        a * a * c * c + a * a * d * d + b * b * c * c + b * b * d * d
} by {
    a * c * a * c = a * a * c * c
    b * d * b * d = b * b * d * d
    a * d * a * d = a * a * d * d
    b * c * b * c = b * b * c * c
    a * a * c * c + b * b * d * d + a * a * d * d + b * b * c * c =
        (a * a * c * c + b * b * d * d) + (a * a * d * d + b * b * c * c)
    real_add_pair_rearrange(a * a * c * c, b * b * d * d, a * a * d * d, b * b * c * c)
    (a * a * c * c + b * b * d * d) + (a * a * d * d + b * b * c * c) =
        (a * a * c * c + a * a * d * d) + (b * b * d * d + b * b * c * c)
    b * b * d * d + b * b * c * c = b * b * c * c + b * b * d * d
    (a * a * c * c + a * a * d * d) + (b * b * d * d + b * b * c * c) =
        (a * a * c * c + a * a * d * d) + (b * b * c * c + b * b * d * d)
    (a * a * c * c + a * a * d * d) + (b * b * c * c + b * b * d * d) =
        a * a * c * c + a * a * d * d + b * b * c * c + b * b * d * d
}

/// The left-hand side of the identity expands to four pure terms.
theorem bf_lhs_expand(a: Real, b: Real, c: Real, d: Real) {
    (a * a + b * b) * (c * c + d * d) =
        a * a * c * c + a * a * d * d + b * b * c * c + b * b * d * d
} by {
    (a * a + b * b) * (c * c + d * d) = (a * a + b * b) * (c * c) + (a * a + b * b) * (d * d)
    (a * a + b * b) * (c * c) = a * a * (c * c) + b * b * (c * c)
    (a * a + b * b) * (d * d) = a * a * (d * d) + b * b * (d * d)
    (a * a + b * b) * (c * c + d * d) =
        a * a * (c * c) + b * b * (c * c) + (a * a * (d * d) + b * b * (d * d))
    a * a * (c * c) + b * b * (c * c) + (a * a * (d * d) + b * b * (d * d)) =
        a * a * (c * c) + (b * b * (c * c) + a * a * (d * d)) + b * b * (d * d)
    b * b * (c * c) + a * a * (d * d) = a * a * (d * d) + b * b * (c * c)
    a * a * (c * c) + (b * b * (c * c) + a * a * (d * d)) + b * b * (d * d) =
        a * a * (c * c) + (a * a * (d * d) + b * b * (c * c)) + b * b * (d * d)
    a * a * (c * c) + (a * a * (d * d) + b * b * (c * c)) + b * b * (d * d) =
        a * a * (c * c) + a * a * (d * d) + b * b * (c * c) + b * b * (d * d)
    a * a * (c * c) + a * a * (d * d) + b * b * (c * c) + b * b * (d * d) =
        a * a * c * c + a * a * d * d + b * b * c * c + b * b * d * d
}

/// The Brahmagupta-Fibonacci identity: the product of two sums of two
/// squares is a sum of two squares,
///     (a² + b²)(c² + d²) = (ac - bd)² + (ad + bc)².
/// The two squares on the right are the norm of the product of the
/// Gaussian integers a + bi and c + di.  Over the integers the identity
/// makes the set of sums of two squares closed under multiplication; the
/// natural-number version must split on whether ac - bd or bd - ac is the
/// nonnegative difference (natural subtraction is truncated), so it is
/// recorded as a statement after the identity.
theorem brahmagupta_fibonacci_identity(a: Real, b: Real, c: Real, d: Real) {
    (a * a + b * b) * (c * c + d * d) =
        (a * c - b * d) * (a * c - b * d) + (a * d + b * c) * (a * d + b * c)
} by {
    bf_square_sub(a * c, b * d)
    (a * c - b * d) * (a * c - b * d) =
        a * c * a * c - a * c * b * d - b * d * a * c + b * d * b * d
    bf_square_add(a * d, b * c)
    (a * d + b * c) * (a * d + b * c) =
        a * d * a * d + b * c * a * d + (a * d * b * c + b * c * b * c)
    a * d * b * c = a * c * b * d
    b * c * a * d = b * d * a * c
    bf_cancel_middle(a * c * a * c, b * d * b * d, a * d * a * d, b * c * b * c,
        a * c * b * d, b * d * a * c)
    a * c * a * c - a * c * b * d - b * d * a * c + b * d * b * d +
        (a * d * a * d + a * c * b * d + b * d * a * c + b * c * b * c) =
        a * c * a * c + b * d * b * d + a * d * a * d + b * c * b * c
    bf_pure_reassemble(a, b, c, d)
    a * c * a * c + b * d * b * d + a * d * a * d + b * c * b * c =
        a * a * c * c + a * a * d * d + b * b * c * c + b * b * d * d
    bf_lhs_expand(a, b, c, d)
    (a * a + b * b) * (c * c + d * d) =
        a * a * c * c + a * a * d * d + b * b * c * c + b * b * d * d
}

// The natural-number closure of the sums of two squares under
// multiplication.  With truncated natural subtraction the identity requires
// a case split: if a·d >= b·c then (a² + b²)(c² + d²) = (ac + bd)² + (ad - bc)²,
// and otherwise the same identity holds with bc - ad in place of ad - bc.
// The algebra of the case split is not yet formalised; the statement is:
//
// theorem sum_two_squares_closed_mul(m: Nat, n: Nat) {
//     is_sum_two_squares(m) and is_sum_two_squares(n) implies is_sum_two_squares(m * n)
// }

// ============================================================================
// Section 7: Lagrange's four-square theorem (statement)
// ============================================================================

/// Lagrange's four-square theorem: every natural number is a sum of four
/// squares.  The library's `four_squares.ac` develops the predicate and the
/// closure machinery, and the concrete values zero through four are sums of
/// four squares.  The theorem itself needs the descent argument: it
/// suffices to prove the claim for primes, where the pigeonhole principle
/// supplies x, y with x² + y² = m·p for some m < p, and the two-square
/// identity descends the representation to a smaller multiple of p; the
/// pigeonhole step is not yet formalised
/// (see theorems1000/theorem_lagrange_four_squares.ac), so the theorem is
/// recorded here as a statement.
///
/// theorem lagrange_four_squares(n: Nat) {
///     is_sum_four_squares(n)
/// }
