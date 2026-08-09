from number_theory.primitive_root import Nat, is_power_of_mod, powers_cover_units_mod,
    powers_cover_units_mod_apply, even_power_quadratic_residue_mod, power_mod_residues,
    full_order_power_mod_residues_unique, full_multiplicative_order_powers_cover_units_mod
from number_theory.multiplicative_order import is_multiplicative_order_mod,
    multiplicative_order_mod, multiplicative_order_mod_pow_to_divisor,
    multiplicative_order_mod_positive, multiplicative_order_mod_minimal,
    multiplicative_order_pow_congr_one, multiplicative_order_mod_is_order,
    powers_below_multiplicative_order_mod_congr_imp_eq, multiplicative_order_mod_not_coprime
from number_theory.totient import totient_prime, coprime_below_prime, totient_pp
from number_theory.congruence import congr_mod_refl, congr_mod_symm, congr_mod_trans,
    congr_mod_pow, congr_mod_mul
from number_theory.coprime import coprime_pow_right, gcd_zero_left
from number_theory.quadratic_residue import is_quadratic_residue_mod,
    is_unit_quadratic_residue_mod, quadratic_residue_coprime_is_unit,
    unit_quadratic_residue_is_residue, unit_quadratic_residue_coprime,
    congr_mod_preserves_coprime, square_coprime_imp_base
from number_theory.zsigmondy import five_is_prime
from number_theory.carmichael import two_is_prime
from nat import exp_mul, exp_add, small_mod, mod_of_decomp, sq_eq_mul,
    lt_suc, lt_imp_lt_suc, lt_suc_right, lt_not_ref, not_lt_zero, trichotomy,
    lt_imp_lte_suc, lte_and_lt, lte_imp_not_lt
from list import List, filter_length_of_pointwise, range_contains_iff_lt, filter_add_length
from data.list.list_filter_count import filter_cons_of_true, filter_cons_of_false
numerals Nat

// ---------------------------------------------------------------------------
// Applications of primitive roots.
//
// Throughout, `g` is a primitive root modulo the prime `p`, formalized as
// `p.is_prime and g.coprime(p) and multiplicative_order_mod(g, p) = p - Nat.1`.
// ---------------------------------------------------------------------------

/// The powers `g^0, ..., g^(p-2)` of a primitive root modulo the prime `p`
/// have pairwise distinct least residues.
theorem primitive_root_power_residues_distinct(p: Nat, g: Nat) {
    p.is_prime and g.coprime(p) and multiplicative_order_mod(g, p) = p - Nat.1
        implies power_mod_residues(g, p).is_unique
} by {
    if p.is_prime and g.coprime(p) and multiplicative_order_mod(g, p) = p - Nat.1 {
        Nat.1 < p
        p != Nat.0
        totient_prime(p)
        p.totient = p - Nat.1
        multiplicative_order_mod(g, p) = p.totient
        full_order_power_mod_residues_unique(g, p)
        power_mod_residues(g, p).is_unique
    }
}

/// Exponents below `p - 1` give distinct powers of a primitive root modulo
/// the prime `p`.
theorem primitive_root_powers_below_order_injective(p: Nat, g: Nat, i: Nat, j: Nat) {
    p.is_prime and g.coprime(p) and multiplicative_order_mod(g, p) = p - Nat.1
        and i < p - Nat.1 and j < p - Nat.1 and g.pow(i).congr_mod(g.pow(j), p)
        implies i = j
} by {
    if p.is_prime and g.coprime(p) and multiplicative_order_mod(g, p) = p - Nat.1
        and i < p - Nat.1 and j < p - Nat.1 and g.pow(i).congr_mod(g.pow(j), p) {
        Nat.1 < p
        p != Nat.0
        p - Nat.1 = multiplicative_order_mod(g, p)
        i < multiplicative_order_mod(g, p)
        j < multiplicative_order_mod(g, p)
        powers_below_multiplicative_order_mod_congr_imp_eq(g, p, i, j)
        i = j
    }
}

/// If `d` divides `p - 1`, the power `g^((p-1)/d)` has multiplicative order
/// exactly `d` modulo the prime `p`.
theorem primitive_root_power_has_order_d(p: Nat, g: Nat, d: Nat) {
    p.is_prime and g.coprime(p) and multiplicative_order_mod(g, p) = p - Nat.1
        and d != Nat.0 and d.divides(p - Nat.1)
        implies multiplicative_order_mod(g.pow((p - Nat.1).div(d)), p) = d
} by {
    if p.is_prime and g.coprime(p) and multiplicative_order_mod(g, p) = p - Nat.1
        and d != Nat.0 and d.divides(p - Nat.1) {
        Nat.1 < p
        p != Nat.0
        d.divides(multiplicative_order_mod(g, p))
        multiplicative_order_mod_pow_to_divisor(g, p, d)
        multiplicative_order_mod(g.pow(multiplicative_order_mod(g, p).div(d)), p) = d
        multiplicative_order_mod(g, p).div(d) = (p - Nat.1).div(d)
        multiplicative_order_mod(g.pow((p - Nat.1).div(d)), p) = d
    }
}

/// Every divisor `d` of `p - 1` is the multiplicative order of some element
/// modulo the prime `p`.
theorem primitive_root_exists_element_of_order_d(p: Nat, g: Nat, d: Nat) {
    p.is_prime and g.coprime(p) and multiplicative_order_mod(g, p) = p - Nat.1
        and d != Nat.0 and d.divides(p - Nat.1)
        implies exists(x: Nat) { x.coprime(p) and multiplicative_order_mod(x, p) = d }
} by {
    if p.is_prime and g.coprime(p) and multiplicative_order_mod(g, p) = p - Nat.1
        and d != Nat.0 and d.divides(p - Nat.1) {
        coprime_pow_right(g, p, (p - Nat.1).div(d))
        g.pow((p - Nat.1).div(d)).coprime(p)
        primitive_root_power_has_order_d(p, g, d)
        multiplicative_order_mod(g.pow((p - Nat.1).div(d)), p) = d
        exists(x: Nat) { x.coprime(p) and multiplicative_order_mod(x, p) = d }
    }
}

/// True when `a` is congruent to a power of `g` with even exponent.
define is_even_power_of_mod(a: Nat, g: Nat, n: Nat) -> Bool {
    exists(k: Nat) { Nat.2.divides(k) and a.congr_mod(g.pow(k), n) }
}

/// Even powers of any base are quadratic residues.
theorem even_power_of_mod_quadratic_residue(a: Nat, g: Nat, p: Nat) {
    is_even_power_of_mod(a, g, p) implies is_quadratic_residue_mod(a, p)
} by {
    if is_even_power_of_mod(a, g, p) {
        is_even_power_of_mod(a, g, p) =
            exists(k: Nat) { Nat.2.divides(k) and a.congr_mod(g.pow(k), p) }
        let k: Nat satisfy { Nat.2.divides(k) and a.congr_mod(g.pow(k), p) }
        even_power_quadratic_residue_mod(g, k, a, p)
        is_quadratic_residue_mod(a, p)
    }
}

/// A unit quadratic residue modulo a prime with primitive root `g` is an even
/// power of `g`.
theorem quadratic_residue_unit_imp_even_power(p: Nat, g: Nat, a: Nat) {
    p.is_prime and g.coprime(p) and multiplicative_order_mod(g, p) = p - Nat.1
        and a.coprime(p) and is_quadratic_residue_mod(a, p)
        implies is_even_power_of_mod(a, g, p)
} by {
    if p.is_prime and g.coprime(p) and multiplicative_order_mod(g, p) = p - Nat.1
        and a.coprime(p) and is_quadratic_residue_mod(a, p) {
        Nat.1 < p
        p != Nat.0
        totient_prime(p)
        p.totient = p - Nat.1
        multiplicative_order_mod(g, p) = p.totient
        full_multiplicative_order_powers_cover_units_mod(g, p)
        powers_cover_units_mod(g, p)
        quadratic_residue_coprime_is_unit(a, p)
        is_unit_quadratic_residue_mod(a, p)
        let x: Nat satisfy { x.coprime(p) and x.pow(Nat.2).congr_mod(a, p) }
        powers_cover_units_mod_apply(g, p, x)
        is_power_of_mod(x, g, p)
        let j: Nat satisfy { x.congr_mod(g.pow(j), p) }
        congr_mod_pow(x, g.pow(j), p, Nat.2)
        x.pow(Nat.2).congr_mod(g.pow(j).pow(Nat.2), p)
        exp_mul(g, j, Nat.2)
        g.pow(j * Nat.2) = g.pow(j).pow(Nat.2)
        g.pow(j).pow(Nat.2) = g.pow(j * Nat.2)
        x.pow(Nat.2).congr_mod(g.pow(j * Nat.2), p)
        congr_mod_symm(x.pow(Nat.2), a, p)
        a.congr_mod(x.pow(Nat.2), p)
        congr_mod_trans(a, x.pow(Nat.2), g.pow(j * Nat.2), p)
        a.congr_mod(g.pow(j * Nat.2), p)
        Nat.2.divides(j * Nat.2)
        Nat.2.divides(j * Nat.2) and a.congr_mod(g.pow(j * Nat.2), p)
        exists(k: Nat) { Nat.2.divides(k) and a.congr_mod(g.pow(k), p) }
        is_even_power_of_mod(a, g, p)
    }
}

/// For a primitive root `g` modulo the prime `p`, the quadratic residues
/// coprime to `p` are exactly the even powers of `g`.
theorem primitive_root_quadratic_residue_iff_even_power(p: Nat, g: Nat, a: Nat) {
    p.is_prime and g.coprime(p) and multiplicative_order_mod(g, p) = p - Nat.1
        and a.coprime(p)
        implies (is_quadratic_residue_mod(a, p) = is_even_power_of_mod(a, g, p))
} by {
    if p.is_prime and g.coprime(p) and multiplicative_order_mod(g, p) = p - Nat.1
        and a.coprime(p) {
        if is_quadratic_residue_mod(a, p) {
            quadratic_residue_unit_imp_even_power(p, g, a)
            is_even_power_of_mod(a, g, p)
        }
        if is_even_power_of_mod(a, g, p) {
            even_power_of_mod_quadratic_residue(a, g, p)
            is_quadratic_residue_mod(a, p)
        }
        (is_quadratic_residue_mod(a, p) = is_even_power_of_mod(a, g, p)) = true
    }
}

// ---------------------------------------------------------------------------
// The number of elements of each order.
//
// The classical statement — for a primitive root `g` modulo the prime `p`,
// exactly `φ(d)` elements of the unit group have multiplicative order `d`,
// for every divisor `d` of `p - 1`:
//
//   theorem primitive_root_order_d_count(p: Nat, g: Nat, d: Nat) {
//       p.is_prime and g.coprime(p) and multiplicative_order_mod(g, p) = p - Nat.1
//           and d != Nat.0 and d.divides(p - Nat.1) implies
//           p.range.filter(function(x: Nat) {
//               multiplicative_order_mod(x, p) = d
//           }).length = d.totient
//   }
//
// is left unproved: it needs the counting identity that the exponents
// `k < p - 1` with `gcd(p - 1, k) = (p - 1) / d` number `φ(d)`, which the
// library does not yet have.  We prove the small case `p = 5` below by
// explicit computation of the orders of the five residues.
// ---------------------------------------------------------------------------

// ---------------------------------------------------------------------------
// Small case: the prime `5`.  The elements of order `d` (for `d` dividing `4`)
// are counted explicitly, matching `φ(d)`.
// ---------------------------------------------------------------------------

/// `1 < 5`.
theorem lt_one_mod_five {
    Nat.1 < Nat.5
} by {
    lt_suc(Nat.1)
    Nat.1 < Nat.2
    lt_imp_lt_suc(Nat.1, Nat.2)
    Nat.1 < Nat.3
    lt_imp_lt_suc(Nat.1, Nat.3)
    Nat.1 < Nat.4
    lt_imp_lt_suc(Nat.1, Nat.4)
    Nat.1 < Nat.5
}

/// `2 < 5`.
theorem lt_two_mod_five {
    Nat.2 < Nat.5
} by {
    lt_suc(Nat.2)
    Nat.2 < Nat.3
    lt_imp_lt_suc(Nat.2, Nat.3)
    Nat.2 < Nat.4
    lt_imp_lt_suc(Nat.2, Nat.4)
    Nat.2 < Nat.5
}

/// `3 < 5`.
theorem lt_three_mod_five {
    Nat.3 < Nat.5
} by {
    lt_suc(Nat.3)
    Nat.3 < Nat.4
    lt_imp_lt_suc(Nat.3, Nat.4)
    Nat.3 < Nat.5
}

/// `0 < 1`.
theorem lt_zero_one_mod_five {
    Nat.0 < Nat.1
} by {
    lt_suc(Nat.0)
}

/// `0 < 2`.
theorem lt_zero_two_mod_five {
    Nat.0 < Nat.2
} by {
    lt_suc(Nat.0)
    Nat.0 < Nat.1
    lt_imp_lt_suc(Nat.0, Nat.1)
    Nat.0 < Nat.2
}

/// `0 < 4`.
theorem lt_zero_four_mod_five {
    Nat.0 < Nat.4
} by {
    lt_suc(Nat.0)
    Nat.0 < Nat.1
    lt_imp_lt_suc(Nat.0, Nat.1)
    Nat.0 < Nat.2
    lt_imp_lt_suc(Nat.0, Nat.2)
    Nat.0 < Nat.3
    lt_imp_lt_suc(Nat.0, Nat.3)
    Nat.0 < Nat.4
}

/// `2` is coprime to `5`.
theorem two_coprime_mod_five {
    Nat.2.coprime(Nat.5)
} by {
    five_is_prime
    Nat.5.is_prime
    Nat.1 <= Nat.2
    lt_two_mod_five
    Nat.2 < Nat.5
    coprime_below_prime(Nat.5, Nat.2)
    Nat.2.coprime(Nat.5)
}

/// `3` is coprime to `5`.
theorem three_coprime_mod_five {
    Nat.3.coprime(Nat.5)
} by {
    five_is_prime
    Nat.5.is_prime
    Nat.1 <= Nat.3
    lt_three_mod_five
    Nat.3 < Nat.5
    coprime_below_prime(Nat.5, Nat.3)
    Nat.3.coprime(Nat.5)
}

/// `4` is coprime to `5`.
theorem four_coprime_mod_five {
    Nat.4.coprime(Nat.5)
} by {
    five_is_prime
    Nat.5.is_prime
    Nat.1 <= Nat.4
    lt_three_mod_five
    Nat.3 < Nat.5
    lt_imp_lt_suc(Nat.3, Nat.4)
    Nat.4 < Nat.5
    coprime_below_prime(Nat.5, Nat.4)
    Nat.4.coprime(Nat.5)
}

/// `0` is not coprime to `5`.
theorem zero_not_coprime_mod_five {
    not Nat.0.coprime(Nat.5)
} by {
    gcd_zero_left(Nat.5)
    Nat.0.gcd(Nat.5) = Nat.5
    if Nat.0.coprime(Nat.5) {
        Nat.0.gcd(Nat.5) = Nat.1
        Nat.5 = Nat.1
        false
    }
}

/// `1.mod(5) = 1`.
theorem mod_one_mod_five {
    Nat.1.mod(Nat.5) = Nat.1
} by {
    lt_one_mod_five
    Nat.1 < Nat.5
    small_mod(Nat.1, Nat.5)
}

/// `2.mod(5) = 2`.
theorem mod_two_mod_five {
    Nat.2.mod(Nat.5) = Nat.2
} by {
    lt_two_mod_five
    Nat.2 < Nat.5
    small_mod(Nat.2, Nat.5)
}

/// `3.mod(5) = 3`.
theorem mod_three_mod_five {
    Nat.3.mod(Nat.5) = Nat.3
} by {
    lt_three_mod_five
    Nat.3 < Nat.5
    small_mod(Nat.3, Nat.5)
}

/// `4.mod(5) = 4`.
theorem mod_four_mod_five {
    Nat.4.mod(Nat.5) = Nat.4
} by {
    lt_three_mod_five
    Nat.3 < Nat.5
    lt_imp_lt_suc(Nat.3, Nat.4)
    Nat.4 < Nat.5
    small_mod(Nat.4, Nat.5)
}

/// `16.mod(5) = 1`.
theorem mod_sixteen_mod_five {
    Nat.16.mod(Nat.5) = Nat.1
} by {
    Nat.3 * Nat.5 = Nat.15
    Nat.15 + Nat.1 = Nat.16
    lt_one_mod_five
    Nat.1 < Nat.5
    mod_of_decomp(Nat.3, Nat.1, Nat.5)
    (Nat.3 * Nat.5 + Nat.1).mod(Nat.5) = Nat.1
    Nat.16.mod(Nat.5) = Nat.1
}

/// `9.mod(5) = 4`.
theorem mod_nine_mod_five {
    Nat.9.mod(Nat.5) = Nat.4
} by {
    Nat.1 * Nat.5 = Nat.5
    Nat.5 + Nat.4 = Nat.9
    lt_three_mod_five
    Nat.3 < Nat.5
    lt_imp_lt_suc(Nat.3, Nat.4)
    Nat.4 < Nat.5
    mod_of_decomp(Nat.1, Nat.4, Nat.5)
    (Nat.1 * Nat.5 + Nat.4).mod(Nat.5) = Nat.4
    Nat.9.mod(Nat.5) = Nat.4
}

/// `12.mod(5) = 2`.
theorem mod_twelve_mod_five {
    Nat.12.mod(Nat.5) = Nat.2
} by {
    Nat.2 * Nat.5 + Nat.2 = Nat.12
    lt_two_mod_five
    Nat.2 < Nat.5
    mod_of_decomp(Nat.2, Nat.2, Nat.5)
    (Nat.2 * Nat.5 + Nat.2).mod(Nat.5) = Nat.2
    Nat.12.mod(Nat.5) = Nat.2
}

/// `2^1 = 2`.
theorem pow_two_one_mod_five {
    Nat.2.pow(Nat.1) = Nat.2
}

/// `2^2 = 4`.
theorem pow_two_two_mod_five {
    Nat.2.pow(Nat.2) = Nat.4
}

/// `2^3 = 8`.
theorem pow_two_three_mod_five {
    Nat.2.pow(Nat.3) = Nat.8
} by {
    exp_add(Nat.2, Nat.2, Nat.1)
    Nat.2.pow(Nat.2 + Nat.1) = Nat.2.pow(Nat.2) * Nat.2.pow(Nat.1)
    Nat.2 + Nat.1 = Nat.3
    Nat.2.pow(Nat.3) = Nat.2.pow(Nat.2) * Nat.2.pow(Nat.1)
    pow_two_two_mod_five
    Nat.2.pow(Nat.2) = Nat.4
    pow_two_one_mod_five
    Nat.2.pow(Nat.1) = Nat.2
    Nat.2.pow(Nat.3) = Nat.4 * Nat.2
    Nat.4 * Nat.2 = Nat.8
    Nat.2.pow(Nat.3) = Nat.8
}

/// `2^4 = 16`.
theorem pow_two_four_mod_five {
    Nat.2.pow(Nat.4) = Nat.16
}

/// `3^1 = 3`.
theorem pow_three_one_mod_five {
    Nat.3.pow(Nat.1) = Nat.3
}

/// `3^2 = 9`.
theorem pow_three_two_mod_five {
    Nat.3.pow(Nat.2) = Nat.9
}

/// `3^3 = 27`.
theorem pow_three_three_mod_five {
    Nat.3.pow(Nat.3) = Nat.27
} by {
    exp_add(Nat.3, Nat.2, Nat.1)
    Nat.3.pow(Nat.2 + Nat.1) = Nat.3.pow(Nat.2) * Nat.3.pow(Nat.1)
    Nat.2 + Nat.1 = Nat.3
    Nat.3.pow(Nat.3) = Nat.3.pow(Nat.2) * Nat.3.pow(Nat.1)
    pow_three_two_mod_five
    Nat.3.pow(Nat.2) = Nat.9
    pow_three_one_mod_five
    Nat.3.pow(Nat.1) = Nat.3
    Nat.3.pow(Nat.3) = Nat.9 * Nat.3
    Nat.9 * Nat.3 = Nat.27
    Nat.3.pow(Nat.3) = Nat.27
}

/// `3^4 = 81`.
theorem pow_three_four_mod_five {
    Nat.3.pow(Nat.4) = Nat.81
} by {
    exp_add(Nat.3, Nat.2, Nat.2)
    Nat.3.pow(Nat.2 + Nat.2) = Nat.3.pow(Nat.2) * Nat.3.pow(Nat.2)
    Nat.2 + Nat.2 = Nat.4
    Nat.3.pow(Nat.4) = Nat.3.pow(Nat.2) * Nat.3.pow(Nat.2)
    pow_three_two_mod_five
    Nat.3.pow(Nat.2) = Nat.9
    Nat.3.pow(Nat.4) = Nat.9 * Nat.9
    Nat.9 * Nat.9 = Nat.81
    Nat.3.pow(Nat.4) = Nat.81
}

/// `4^1 = 4`.
theorem pow_four_one_mod_five {
    Nat.4.pow(Nat.1) = Nat.4
}

/// `4^2 = 16`.
theorem pow_four_two_mod_five {
    Nat.4.pow(Nat.2) = Nat.16
}

/// `1^1 = 1`.
theorem pow_one_one_mod_five {
    Nat.1.pow(Nat.1) = Nat.1
}

/// `9 * 3 = 27`.
theorem mul_nine_three_mod_five {
    Nat.9 * Nat.3 = Nat.27
}

/// `4 * 3 = 12`.
theorem mul_four_three_mod_five {
    Nat.4 * Nat.3 = Nat.12
}

/// `16 ≡ 1 (mod 5)`.
theorem congr_sixteen_mod_five {
    Nat.16.congr_mod(Nat.1, Nat.5)
} by {
    mod_sixteen_mod_five
    Nat.16.mod(Nat.5) = Nat.1
    mod_one_mod_five
    Nat.1.mod(Nat.5) = Nat.1
    Nat.16.mod(Nat.5) = Nat.1.mod(Nat.5)
    Nat.16.congr_mod(Nat.1, Nat.5)
}

/// `9 ≡ 4 (mod 5)`.
theorem congr_nine_mod_five {
    Nat.9.congr_mod(Nat.4, Nat.5)
} by {
    mod_nine_mod_five
    Nat.9.mod(Nat.5) = Nat.4
    mod_four_mod_five
    Nat.4.mod(Nat.5) = Nat.4
    Nat.9.mod(Nat.5) = Nat.4.mod(Nat.5)
    Nat.9.congr_mod(Nat.4, Nat.5)
}

/// `12 ≡ 2 (mod 5)`.
theorem congr_twelve_mod_five {
    Nat.12.congr_mod(Nat.2, Nat.5)
} by {
    mod_twelve_mod_five
    Nat.12.mod(Nat.5) = Nat.2
    mod_two_mod_five
    Nat.2.mod(Nat.5) = Nat.2
    Nat.12.mod(Nat.5) = Nat.2.mod(Nat.5)
    Nat.12.congr_mod(Nat.2, Nat.5)
}

/// `81 ≡ 1 (mod 5)`.
theorem congr_eighty_one_mod_five {
    Nat.81.congr_mod(Nat.1, Nat.5)
} by {
    congr_nine_mod_five
    Nat.9.congr_mod(Nat.4, Nat.5)
    congr_mod_mul(Nat.9, Nat.9, Nat.4, Nat.4, Nat.5)
    (Nat.9 * Nat.9).congr_mod(Nat.4 * Nat.4, Nat.5)
    Nat.9 * Nat.9 = Nat.81
    Nat.4 * Nat.4 = Nat.16
    Nat.81.congr_mod(Nat.16, Nat.5)
    congr_sixteen_mod_five
    Nat.16.congr_mod(Nat.1, Nat.5)
    congr_mod_trans(Nat.81, Nat.16, Nat.1, Nat.5)
    Nat.81.congr_mod(Nat.1, Nat.5)
}

/// `2 ≢ 1 (mod 5)`.
theorem not_congr_two_mod_five {
    not Nat.2.congr_mod(Nat.1, Nat.5)
} by {
    mod_two_mod_five
    Nat.2.mod(Nat.5) = Nat.2
    mod_one_mod_five
    Nat.1.mod(Nat.5) = Nat.1
    if Nat.2.congr_mod(Nat.1, Nat.5) {
        Nat.2.mod(Nat.5) = Nat.1.mod(Nat.5)
        Nat.2 = Nat.1
        false
    }
}

/// `3 ≢ 1 (mod 5)`.
theorem not_congr_three_mod_five {
    not Nat.3.congr_mod(Nat.1, Nat.5)
} by {
    mod_three_mod_five
    Nat.3.mod(Nat.5) = Nat.3
    mod_one_mod_five
    Nat.1.mod(Nat.5) = Nat.1
    if Nat.3.congr_mod(Nat.1, Nat.5) {
        Nat.3.mod(Nat.5) = Nat.1.mod(Nat.5)
        Nat.3 = Nat.1
        false
    }
}

/// `4 ≢ 1 (mod 5)`.
theorem not_congr_four_mod_five {
    not Nat.4.congr_mod(Nat.1, Nat.5)
} by {
    mod_four_mod_five
    Nat.4.mod(Nat.5) = Nat.4
    mod_one_mod_five
    Nat.1.mod(Nat.5) = Nat.1
    if Nat.4.congr_mod(Nat.1, Nat.5) {
        Nat.4.mod(Nat.5) = Nat.1.mod(Nat.5)
        Nat.4 = Nat.1
        false
    }
}

/// `9 ≢ 1 (mod 5)`.
theorem not_congr_nine_mod_five {
    not Nat.9.congr_mod(Nat.1, Nat.5)
} by {
    mod_nine_mod_five
    Nat.9.mod(Nat.5) = Nat.4
    mod_one_mod_five
    Nat.1.mod(Nat.5) = Nat.1
    if Nat.9.congr_mod(Nat.1, Nat.5) {
        Nat.9.mod(Nat.5) = Nat.1.mod(Nat.5)
        Nat.4 = Nat.1
        false
    }
}

/// `27 ≢ 1 (mod 5)`.
theorem not_congr_twenty_seven_mod_five {
    not Nat.27.congr_mod(Nat.1, Nat.5)
} by {
    // 27 = 9 * 3 ≡ 4 * 3 = 12 ≡ 2 (mod 5), and 2 ≢ 1.
    congr_nine_mod_five
    Nat.9.congr_mod(Nat.4, Nat.5)
    congr_mod_refl(Nat.3, Nat.5)
    Nat.3.congr_mod(Nat.3, Nat.5)
    congr_mod_mul(Nat.9, Nat.3, Nat.4, Nat.3, Nat.5)
    (Nat.9 * Nat.3).congr_mod(Nat.4 * Nat.3, Nat.5)
    mul_nine_three_mod_five
    Nat.9 * Nat.3 = Nat.27
    mul_four_three_mod_five
    Nat.4 * Nat.3 = Nat.12
    Nat.27.congr_mod(Nat.12, Nat.5)
    congr_twelve_mod_five
    Nat.12.congr_mod(Nat.2, Nat.5)
    congr_mod_trans(Nat.27, Nat.12, Nat.2, Nat.5)
    Nat.27.congr_mod(Nat.2, Nat.5)
    if Nat.27.congr_mod(Nat.1, Nat.5) {
        congr_mod_symm(Nat.27, Nat.2, Nat.5)
        Nat.2.congr_mod(Nat.27, Nat.5)
        congr_mod_trans(Nat.2, Nat.27, Nat.1, Nat.5)
        Nat.2.congr_mod(Nat.1, Nat.5)
        not_congr_two_mod_five
        false
    }
}

/// `2^1 ≢ 1 (mod 5)`.
theorem not_congr_two_pow_one_mod_five {
    not Nat.2.pow(Nat.1).congr_mod(Nat.1, Nat.5)
} by {
    pow_two_one_mod_five
    Nat.2.pow(Nat.1) = Nat.2
    not_congr_two_mod_five
    not Nat.2.congr_mod(Nat.1, Nat.5)
    if Nat.2.pow(Nat.1).congr_mod(Nat.1, Nat.5) {
        Nat.2.congr_mod(Nat.1, Nat.5)
        false
    }
}

/// `2^2 ≢ 1 (mod 5)`.
theorem not_congr_two_pow_two_mod_five {
    not Nat.2.pow(Nat.2).congr_mod(Nat.1, Nat.5)
} by {
    pow_two_two_mod_five
    Nat.2.pow(Nat.2) = Nat.4
    not_congr_four_mod_five
    not Nat.4.congr_mod(Nat.1, Nat.5)
    if Nat.2.pow(Nat.2).congr_mod(Nat.1, Nat.5) {
        Nat.4.congr_mod(Nat.1, Nat.5)
        false
    }
}

/// `2^3 ≢ 1 (mod 5)`.
theorem not_congr_two_pow_three_mod_five {
    not Nat.2.pow(Nat.3).congr_mod(Nat.1, Nat.5)
} by {
    pow_two_three_mod_five
    Nat.2.pow(Nat.3) = Nat.8
    // 8 = 1 * 5 + 3, so 8 ≡ 3 (mod 5); and 3 ≢ 1.
    Nat.1 * Nat.5 = Nat.5
    Nat.5 + Nat.3 = Nat.8
    lt_three_mod_five
    Nat.3 < Nat.5
    mod_of_decomp(Nat.1, Nat.3, Nat.5)
    (Nat.1 * Nat.5 + Nat.3).mod(Nat.5) = Nat.3
    Nat.8.mod(Nat.5) = Nat.3
    mod_three_mod_five
    Nat.3.mod(Nat.5) = Nat.3
    Nat.8.mod(Nat.5) = Nat.3.mod(Nat.5)
    Nat.8.congr_mod(Nat.3, Nat.5)
    not_congr_three_mod_five
    if Nat.8.congr_mod(Nat.1, Nat.5) {
        congr_mod_symm(Nat.8, Nat.3, Nat.5)
        Nat.3.congr_mod(Nat.8, Nat.5)
        congr_mod_trans(Nat.3, Nat.8, Nat.1, Nat.5)
        Nat.3.congr_mod(Nat.1, Nat.5)
        false
    }
    if Nat.2.pow(Nat.3).congr_mod(Nat.1, Nat.5) {
        Nat.8.congr_mod(Nat.1, Nat.5)
        false
    }
}

/// `2^4 ≡ 1 (mod 5)`.
theorem congr_two_pow_four_mod_five {
    Nat.2.pow(Nat.4).congr_mod(Nat.1, Nat.5)
} by {
    pow_two_four_mod_five
    Nat.2.pow(Nat.4) = Nat.16
    congr_sixteen_mod_five
    Nat.16.congr_mod(Nat.1, Nat.5)
    Nat.2.pow(Nat.4).congr_mod(Nat.1, Nat.5)
}

/// `3^1 ≢ 1 (mod 5)`.
theorem not_congr_three_pow_one_mod_five {
    not Nat.3.pow(Nat.1).congr_mod(Nat.1, Nat.5)
} by {
    pow_three_one_mod_five
    Nat.3.pow(Nat.1) = Nat.3
    not_congr_three_mod_five
    not Nat.3.congr_mod(Nat.1, Nat.5)
    if Nat.3.pow(Nat.1).congr_mod(Nat.1, Nat.5) {
        Nat.3.congr_mod(Nat.1, Nat.5)
        false
    }
}

/// `3^2 ≢ 1 (mod 5)`.
theorem not_congr_three_pow_two_mod_five {
    not Nat.3.pow(Nat.2).congr_mod(Nat.1, Nat.5)
} by {
    pow_three_two_mod_five
    Nat.3.pow(Nat.2) = Nat.9
    not_congr_nine_mod_five
    not Nat.9.congr_mod(Nat.1, Nat.5)
    if Nat.3.pow(Nat.2).congr_mod(Nat.1, Nat.5) {
        Nat.9.congr_mod(Nat.1, Nat.5)
        false
    }
}

/// `3^3 ≢ 1 (mod 5)`.
theorem not_congr_three_pow_three_mod_five {
    not Nat.3.pow(Nat.3).congr_mod(Nat.1, Nat.5)
} by {
    pow_three_three_mod_five
    Nat.3.pow(Nat.3) = Nat.27
    not_congr_twenty_seven_mod_five
    not Nat.27.congr_mod(Nat.1, Nat.5)
    if Nat.3.pow(Nat.3).congr_mod(Nat.1, Nat.5) {
        Nat.27.congr_mod(Nat.1, Nat.5)
        false
    }
}

/// `3^4 ≡ 1 (mod 5)`.
theorem congr_three_pow_four_mod_five {
    Nat.3.pow(Nat.4).congr_mod(Nat.1, Nat.5)
} by {
    pow_three_four_mod_five
    Nat.3.pow(Nat.4) = Nat.81
    congr_eighty_one_mod_five
    Nat.81.congr_mod(Nat.1, Nat.5)
    Nat.3.pow(Nat.4).congr_mod(Nat.1, Nat.5)
}

/// `4^1 ≢ 1 (mod 5)`.
theorem not_congr_four_pow_one_mod_five {
    not Nat.4.pow(Nat.1).congr_mod(Nat.1, Nat.5)
} by {
    pow_four_one_mod_five
    Nat.4.pow(Nat.1) = Nat.4
    not_congr_four_mod_five
    not Nat.4.congr_mod(Nat.1, Nat.5)
    if Nat.4.pow(Nat.1).congr_mod(Nat.1, Nat.5) {
        Nat.4.congr_mod(Nat.1, Nat.5)
        false
    }
}

/// `4^2 ≡ 1 (mod 5)`.
theorem congr_four_pow_two_mod_five {
    Nat.4.pow(Nat.2).congr_mod(Nat.1, Nat.5)
} by {
    pow_four_two_mod_five
    Nat.4.pow(Nat.2) = Nat.16
    congr_sixteen_mod_five
    Nat.16.congr_mod(Nat.1, Nat.5)
    Nat.4.pow(Nat.2).congr_mod(Nat.1, Nat.5)
}

/// `1^1 ≡ 1 (mod 5)`.
theorem congr_one_pow_one_mod_five {
    Nat.1.pow(Nat.1).congr_mod(Nat.1, Nat.5)
} by {
    pow_one_one_mod_five
    Nat.1.pow(Nat.1) = Nat.1
    congr_mod_refl(Nat.1, Nat.5)
    Nat.1.congr_mod(Nat.1, Nat.5)
    Nat.1.pow(Nat.1).congr_mod(Nat.1, Nat.5)
}

/// `0` has multiplicative order `0` modulo `5`.
theorem order_zero_mod_five {
    multiplicative_order_mod(Nat.0, Nat.5) = Nat.0
} by {
    zero_not_coprime_mod_five
    not Nat.0.coprime(Nat.5)
    multiplicative_order_mod_not_coprime(Nat.0, Nat.5)
    multiplicative_order_mod(Nat.0, Nat.5) = Nat.0
}

/// `1` has multiplicative order `1` modulo `5`.
theorem order_one_mod_five {
    multiplicative_order_mod(Nat.1, Nat.5) = Nat.1
} by {
    Nat.5 != Nat.0
    Nat.1.coprime(Nat.5)
    lt_zero_one_mod_five
    Nat.0 < Nat.1
    congr_one_pow_one_mod_five
    Nat.1.pow(Nat.1).congr_mod(Nat.1, Nat.5)
    multiplicative_order_mod_minimal(Nat.1, Nat.5, Nat.1)
    multiplicative_order_mod(Nat.1, Nat.5) <= Nat.1
    multiplicative_order_mod_positive(Nat.1, Nat.5)
    Nat.0 < multiplicative_order_mod(Nat.1, Nat.5)
    trichotomy(multiplicative_order_mod(Nat.1, Nat.5), Nat.1)
    if multiplicative_order_mod(Nat.1, Nat.5) < Nat.1 {
        lt_suc_right(multiplicative_order_mod(Nat.1, Nat.5), Nat.0)
        if multiplicative_order_mod(Nat.1, Nat.5) = Nat.0 {
            Nat.0 < multiplicative_order_mod(Nat.1, Nat.5)
            Nat.0 < Nat.0
            lt_not_ref(Nat.0)
            false
        }
        multiplicative_order_mod(Nat.1, Nat.5) < Nat.0
        not_lt_zero(multiplicative_order_mod(Nat.1, Nat.5))
        false
    }
    if Nat.1 < multiplicative_order_mod(Nat.1, Nat.5) {
        lte_imp_not_lt(multiplicative_order_mod(Nat.1, Nat.5), Nat.1)
        not Nat.1 < multiplicative_order_mod(Nat.1, Nat.5)
        false
    }
    multiplicative_order_mod(Nat.1, Nat.5) = Nat.1
}

/// `2` has multiplicative order `4` modulo `5`.
theorem order_two_mod_five {
    multiplicative_order_mod(Nat.2, Nat.5) = Nat.4
} by {
    Nat.5 != Nat.0
    two_coprime_mod_five
    lt_zero_four_mod_five
    Nat.0 < Nat.4
    congr_two_pow_four_mod_five
    Nat.2.pow(Nat.4).congr_mod(Nat.1, Nat.5)
    multiplicative_order_mod_minimal(Nat.2, Nat.5, Nat.4)
    multiplicative_order_mod(Nat.2, Nat.5) <= Nat.4
    multiplicative_order_mod_positive(Nat.2, Nat.5)
    Nat.0 < multiplicative_order_mod(Nat.2, Nat.5)
    if multiplicative_order_mod(Nat.2, Nat.5) = Nat.1 {
        multiplicative_order_mod_is_order(Nat.2, Nat.5)
        is_multiplicative_order_mod(Nat.2, Nat.5, multiplicative_order_mod(Nat.2, Nat.5))
        is_multiplicative_order_mod(Nat.2, Nat.5, Nat.1)
        multiplicative_order_pow_congr_one(Nat.2, Nat.5, Nat.1)
        Nat.2.pow(Nat.1).congr_mod(Nat.1, Nat.5)
        not_congr_two_pow_one_mod_five
        false
    }
    if multiplicative_order_mod(Nat.2, Nat.5) = Nat.2 {
        multiplicative_order_mod_is_order(Nat.2, Nat.5)
        is_multiplicative_order_mod(Nat.2, Nat.5, Nat.2)
        multiplicative_order_pow_congr_one(Nat.2, Nat.5, Nat.2)
        Nat.2.pow(Nat.2).congr_mod(Nat.1, Nat.5)
        not_congr_two_pow_two_mod_five
        false
    }
    if multiplicative_order_mod(Nat.2, Nat.5) = Nat.3 {
        multiplicative_order_mod_is_order(Nat.2, Nat.5)
        is_multiplicative_order_mod(Nat.2, Nat.5, Nat.3)
        multiplicative_order_pow_congr_one(Nat.2, Nat.5, Nat.3)
        Nat.2.pow(Nat.3).congr_mod(Nat.1, Nat.5)
        not_congr_two_pow_three_mod_five
        false
    }
    trichotomy(multiplicative_order_mod(Nat.2, Nat.5), Nat.4)
    if multiplicative_order_mod(Nat.2, Nat.5) < Nat.4 {
        lt_suc_right(multiplicative_order_mod(Nat.2, Nat.5), Nat.3)
        if multiplicative_order_mod(Nat.2, Nat.5) = Nat.3 {
            false
        }
        multiplicative_order_mod(Nat.2, Nat.5) < Nat.3
        lt_suc_right(multiplicative_order_mod(Nat.2, Nat.5), Nat.2)
        if multiplicative_order_mod(Nat.2, Nat.5) = Nat.2 {
            false
        }
        multiplicative_order_mod(Nat.2, Nat.5) < Nat.2
        lt_suc_right(multiplicative_order_mod(Nat.2, Nat.5), Nat.1)
        if multiplicative_order_mod(Nat.2, Nat.5) = Nat.1 {
            false
        }
        multiplicative_order_mod(Nat.2, Nat.5) < Nat.1
        lt_suc_right(multiplicative_order_mod(Nat.2, Nat.5), Nat.0)
        if multiplicative_order_mod(Nat.2, Nat.5) = Nat.0 {
            Nat.0 < multiplicative_order_mod(Nat.2, Nat.5)
            Nat.0 < Nat.0
            lt_not_ref(Nat.0)
            false
        }
        multiplicative_order_mod(Nat.2, Nat.5) < Nat.0
        not_lt_zero(multiplicative_order_mod(Nat.2, Nat.5))
        false
    }
    if Nat.4 < multiplicative_order_mod(Nat.2, Nat.5) {
        lte_imp_not_lt(multiplicative_order_mod(Nat.2, Nat.5), Nat.4)
        not Nat.4 < multiplicative_order_mod(Nat.2, Nat.5)
        false
    }
    multiplicative_order_mod(Nat.2, Nat.5) = Nat.4
}

/// `3` has multiplicative order `4` modulo `5`.
theorem order_three_mod_five {
    multiplicative_order_mod(Nat.3, Nat.5) = Nat.4
} by {
    Nat.5 != Nat.0
    three_coprime_mod_five
    lt_zero_four_mod_five
    Nat.0 < Nat.4
    congr_three_pow_four_mod_five
    Nat.3.pow(Nat.4).congr_mod(Nat.1, Nat.5)
    multiplicative_order_mod_minimal(Nat.3, Nat.5, Nat.4)
    multiplicative_order_mod(Nat.3, Nat.5) <= Nat.4
    multiplicative_order_mod_positive(Nat.3, Nat.5)
    Nat.0 < multiplicative_order_mod(Nat.3, Nat.5)
    if multiplicative_order_mod(Nat.3, Nat.5) = Nat.1 {
        multiplicative_order_mod_is_order(Nat.3, Nat.5)
        is_multiplicative_order_mod(Nat.3, Nat.5, multiplicative_order_mod(Nat.3, Nat.5))
        is_multiplicative_order_mod(Nat.3, Nat.5, Nat.1)
        multiplicative_order_pow_congr_one(Nat.3, Nat.5, Nat.1)
        Nat.3.pow(Nat.1).congr_mod(Nat.1, Nat.5)
        not_congr_three_pow_one_mod_five
        false
    }
    if multiplicative_order_mod(Nat.3, Nat.5) = Nat.2 {
        multiplicative_order_mod_is_order(Nat.3, Nat.5)
        is_multiplicative_order_mod(Nat.3, Nat.5, Nat.2)
        multiplicative_order_pow_congr_one(Nat.3, Nat.5, Nat.2)
        Nat.3.pow(Nat.2).congr_mod(Nat.1, Nat.5)
        not_congr_three_pow_two_mod_five
        false
    }
    if multiplicative_order_mod(Nat.3, Nat.5) = Nat.3 {
        multiplicative_order_mod_is_order(Nat.3, Nat.5)
        is_multiplicative_order_mod(Nat.3, Nat.5, Nat.3)
        multiplicative_order_pow_congr_one(Nat.3, Nat.5, Nat.3)
        Nat.3.pow(Nat.3).congr_mod(Nat.1, Nat.5)
        not_congr_three_pow_three_mod_five
        false
    }
    trichotomy(multiplicative_order_mod(Nat.3, Nat.5), Nat.4)
    if multiplicative_order_mod(Nat.3, Nat.5) < Nat.4 {
        lt_suc_right(multiplicative_order_mod(Nat.3, Nat.5), Nat.3)
        if multiplicative_order_mod(Nat.3, Nat.5) = Nat.3 {
            false
        }
        multiplicative_order_mod(Nat.3, Nat.5) < Nat.3
        lt_suc_right(multiplicative_order_mod(Nat.3, Nat.5), Nat.2)
        if multiplicative_order_mod(Nat.3, Nat.5) = Nat.2 {
            false
        }
        multiplicative_order_mod(Nat.3, Nat.5) < Nat.2
        lt_suc_right(multiplicative_order_mod(Nat.3, Nat.5), Nat.1)
        if multiplicative_order_mod(Nat.3, Nat.5) = Nat.1 {
            false
        }
        multiplicative_order_mod(Nat.3, Nat.5) < Nat.1
        lt_suc_right(multiplicative_order_mod(Nat.3, Nat.5), Nat.0)
        if multiplicative_order_mod(Nat.3, Nat.5) = Nat.0 {
            Nat.0 < multiplicative_order_mod(Nat.3, Nat.5)
            Nat.0 < Nat.0
            lt_not_ref(Nat.0)
            false
        }
        multiplicative_order_mod(Nat.3, Nat.5) < Nat.0
        not_lt_zero(multiplicative_order_mod(Nat.3, Nat.5))
        false
    }
    if Nat.4 < multiplicative_order_mod(Nat.3, Nat.5) {
        lte_imp_not_lt(multiplicative_order_mod(Nat.3, Nat.5), Nat.4)
        not Nat.4 < multiplicative_order_mod(Nat.3, Nat.5)
        false
    }
    multiplicative_order_mod(Nat.3, Nat.5) = Nat.4
}

/// `4` has multiplicative order `2` modulo `5`.
theorem order_four_mod_five {
    multiplicative_order_mod(Nat.4, Nat.5) = Nat.2
} by {
    Nat.5 != Nat.0
    four_coprime_mod_five
    lt_zero_two_mod_five
    Nat.0 < Nat.2
    congr_four_pow_two_mod_five
    Nat.4.pow(Nat.2).congr_mod(Nat.1, Nat.5)
    multiplicative_order_mod_minimal(Nat.4, Nat.5, Nat.2)
    multiplicative_order_mod(Nat.4, Nat.5) <= Nat.2
    multiplicative_order_mod_positive(Nat.4, Nat.5)
    Nat.0 < multiplicative_order_mod(Nat.4, Nat.5)
    if multiplicative_order_mod(Nat.4, Nat.5) = Nat.1 {
        multiplicative_order_mod_is_order(Nat.4, Nat.5)
        is_multiplicative_order_mod(Nat.4, Nat.5, multiplicative_order_mod(Nat.4, Nat.5))
        is_multiplicative_order_mod(Nat.4, Nat.5, Nat.1)
        multiplicative_order_pow_congr_one(Nat.4, Nat.5, Nat.1)
        Nat.4.pow(Nat.1).congr_mod(Nat.1, Nat.5)
        not_congr_four_pow_one_mod_five
        false
    }
    trichotomy(multiplicative_order_mod(Nat.4, Nat.5), Nat.2)
    if multiplicative_order_mod(Nat.4, Nat.5) < Nat.2 {
        lt_suc_right(multiplicative_order_mod(Nat.4, Nat.5), Nat.1)
        if multiplicative_order_mod(Nat.4, Nat.5) = Nat.1 {
            false
        }
        multiplicative_order_mod(Nat.4, Nat.5) < Nat.1
        lt_suc_right(multiplicative_order_mod(Nat.4, Nat.5), Nat.0)
        if multiplicative_order_mod(Nat.4, Nat.5) = Nat.0 {
            Nat.0 < multiplicative_order_mod(Nat.4, Nat.5)
            Nat.0 < Nat.0
            lt_not_ref(Nat.0)
            false
        }
        multiplicative_order_mod(Nat.4, Nat.5) < Nat.0
        not_lt_zero(multiplicative_order_mod(Nat.4, Nat.5))
        false
    }
    if Nat.2 < multiplicative_order_mod(Nat.4, Nat.5) {
        lte_imp_not_lt(multiplicative_order_mod(Nat.4, Nat.5), Nat.2)
        not Nat.2 < multiplicative_order_mod(Nat.4, Nat.5)
        false
    }
    multiplicative_order_mod(Nat.4, Nat.5) = Nat.2
}

// ---------------------------------------------------------------------------
// Totient values for the divisors of 4.
// ---------------------------------------------------------------------------

/// `φ(1) = 1`.
theorem totient_one_mod_five {
    Nat.1.totient = Nat.1
}

/// `φ(2) = 1`.
theorem totient_two_mod_five {
    Nat.2.totient = Nat.1
} by {
    two_is_prime
    Nat.2.is_prime
    totient_prime(Nat.2)
    Nat.2.totient = Nat.2 - Nat.1
}

/// `φ(4) = 2`.
theorem totient_four_mod_five {
    Nat.4.totient = Nat.2
} by {
    two_is_prime
    Nat.2.is_prime
    totient_pp(Nat.2)
    (Nat.2 * Nat.2).totient = Nat.2 * Nat.2 - Nat.2
    Nat.2 * Nat.2 = Nat.4
    Nat.4.totient = Nat.4 - Nat.2
    Nat.4 - Nat.2 = Nat.2
    Nat.4.totient = Nat.2
}


/// The order-`4` residues modulo `5`.
define order_four_pred_mod_five(x: Nat) -> Bool {
    multiplicative_order_mod(x, Nat.5) = Nat.4
}

/// `5.range = [0,1,2,3,4]` in append form.
theorem range_five_mod_five {
    Nat.5.range = Nat.4.range + List.singleton(Nat.4)
}

theorem range_four_mod_five {
    Nat.4.range = Nat.3.range + List.singleton(Nat.3)
}

theorem range_three_mod_five {
    Nat.3.range = Nat.2.range + List.singleton(Nat.2)
}

theorem range_two_mod_five {
    Nat.2.range = Nat.1.range + List.singleton(Nat.1)
}

theorem range_one_mod_five {
    Nat.1.range = Nat.0.range + List.singleton(Nat.0)
}

theorem range_zero_mod_five {
    Nat.0.range = List.nil[Nat]
}

/// `[4]` contains no order-`4` residue.
theorem singleton_order_four_len_4 {
    List.singleton(Nat.4).filter(order_four_pred_mod_five).length = Nat.0
} by {
    order_four_mod_five
    multiplicative_order_mod(Nat.4, Nat.5) = Nat.2
    if order_four_pred_mod_five(Nat.4) {
        multiplicative_order_mod(Nat.4, Nat.5) = Nat.4
        Nat.2 = Nat.4
        false
    }
    not order_four_pred_mod_five(Nat.4)
    filter_cons_of_false(Nat.4, List.nil[Nat], order_four_pred_mod_five)
    List.cons(Nat.4, List.nil[Nat]).filter(order_four_pred_mod_five) =
        List.nil[Nat].filter(order_four_pred_mod_five)
    List.singleton(Nat.4) = List.cons(Nat.4, List.nil[Nat])
    List.singleton(Nat.4).filter(order_four_pred_mod_five) =
        List.nil[Nat].filter(order_four_pred_mod_five)
    List.nil[Nat].filter(order_four_pred_mod_five) = List.nil[Nat]
    List.singleton(Nat.4).filter(order_four_pred_mod_five) = List.nil[Nat]
    List.singleton(Nat.4).filter(order_four_pred_mod_five).length = Nat.0
}

/// `[3]` contains one order-`4` residue.
theorem singleton_order_four_len_3 {
    List.singleton(Nat.3).filter(order_four_pred_mod_five).length = Nat.1
} by {
    order_three_mod_five
    multiplicative_order_mod(Nat.3, Nat.5) = Nat.4
    order_four_pred_mod_five(Nat.3)
    filter_cons_of_true(Nat.3, List.nil[Nat], order_four_pred_mod_five)
    List.cons(Nat.3, List.nil[Nat]).filter(order_four_pred_mod_five) =
        List.cons(Nat.3, List.nil[Nat].filter(order_four_pred_mod_five))
    List.singleton(Nat.3) = List.cons(Nat.3, List.nil[Nat])
    List.singleton(Nat.3).filter(order_four_pred_mod_five) =
        List.cons(Nat.3, List.nil[Nat].filter(order_four_pred_mod_five))
    List.nil[Nat].filter(order_four_pred_mod_five) = List.nil[Nat]
    List.singleton(Nat.3).filter(order_four_pred_mod_five) = List.singleton(Nat.3)
    List.singleton(Nat.3).length = Nat.1
    List.singleton(Nat.3).filter(order_four_pred_mod_five).length = Nat.1
}

/// `[2]` contains one order-`4` residue.
theorem singleton_order_four_len_2 {
    List.singleton(Nat.2).filter(order_four_pred_mod_five).length = Nat.1
} by {
    order_two_mod_five
    multiplicative_order_mod(Nat.2, Nat.5) = Nat.4
    order_four_pred_mod_five(Nat.2)
    filter_cons_of_true(Nat.2, List.nil[Nat], order_four_pred_mod_five)
    List.cons(Nat.2, List.nil[Nat]).filter(order_four_pred_mod_five) =
        List.cons(Nat.2, List.nil[Nat].filter(order_four_pred_mod_five))
    List.singleton(Nat.2) = List.cons(Nat.2, List.nil[Nat])
    List.singleton(Nat.2).filter(order_four_pred_mod_five) =
        List.cons(Nat.2, List.nil[Nat].filter(order_four_pred_mod_five))
    List.nil[Nat].filter(order_four_pred_mod_five) = List.nil[Nat]
    List.singleton(Nat.2).filter(order_four_pred_mod_five) = List.singleton(Nat.2)
    List.singleton(Nat.2).length = Nat.1
    List.singleton(Nat.2).filter(order_four_pred_mod_five).length = Nat.1
}

/// `[1]` contains no order-`4` residue.
theorem singleton_order_four_len_1 {
    List.singleton(Nat.1).filter(order_four_pred_mod_five).length = Nat.0
} by {
    order_one_mod_five
    multiplicative_order_mod(Nat.1, Nat.5) = Nat.1
    if order_four_pred_mod_five(Nat.1) {
        multiplicative_order_mod(Nat.1, Nat.5) = Nat.4
        Nat.1 = Nat.4
        false
    }
    not order_four_pred_mod_five(Nat.1)
    filter_cons_of_false(Nat.1, List.nil[Nat], order_four_pred_mod_five)
    List.cons(Nat.1, List.nil[Nat]).filter(order_four_pred_mod_five) =
        List.nil[Nat].filter(order_four_pred_mod_five)
    List.singleton(Nat.1) = List.cons(Nat.1, List.nil[Nat])
    List.singleton(Nat.1).filter(order_four_pred_mod_five) =
        List.nil[Nat].filter(order_four_pred_mod_five)
    List.nil[Nat].filter(order_four_pred_mod_five) = List.nil[Nat]
    List.singleton(Nat.1).filter(order_four_pred_mod_five) = List.nil[Nat]
    List.singleton(Nat.1).filter(order_four_pred_mod_five).length = Nat.0
}

/// `[0]` contains no order-`4` residue.
theorem singleton_order_four_len_0 {
    List.singleton(Nat.0).filter(order_four_pred_mod_five).length = Nat.0
} by {
    order_zero_mod_five
    multiplicative_order_mod(Nat.0, Nat.5) = Nat.0
    if order_four_pred_mod_five(Nat.0) {
        multiplicative_order_mod(Nat.0, Nat.5) = Nat.4
        Nat.0 = Nat.4
        false
    }
    not order_four_pred_mod_five(Nat.0)
    filter_cons_of_false(Nat.0, List.nil[Nat], order_four_pred_mod_five)
    List.cons(Nat.0, List.nil[Nat]).filter(order_four_pred_mod_five) =
        List.nil[Nat].filter(order_four_pred_mod_five)
    List.singleton(Nat.0) = List.cons(Nat.0, List.nil[Nat])
    List.singleton(Nat.0).filter(order_four_pred_mod_five) =
        List.nil[Nat].filter(order_four_pred_mod_five)
    List.nil[Nat].filter(order_four_pred_mod_five) = List.nil[Nat]
    List.singleton(Nat.0).filter(order_four_pred_mod_five) = List.nil[Nat]
    List.singleton(Nat.0).filter(order_four_pred_mod_five).length = Nat.0
}


/// Exactly `φ(4) = 2` elements have multiplicative order `4` modulo `5`.
theorem count_order_four_mod_five {
    Nat.5.range.filter(order_four_pred_mod_five).length = Nat.2
} by {
    // 5.range = 4.range + [4]
    range_five_mod_five
    Nat.5.range = Nat.4.range + List.singleton(Nat.4)
    filter_add_length(Nat.4.range, List.singleton(Nat.4), order_four_pred_mod_five)
    Nat.5.range.filter(order_four_pred_mod_five).length =
        Nat.4.range.filter(order_four_pred_mod_five).length +
            List.singleton(Nat.4).filter(order_four_pred_mod_five).length
    singleton_order_four_len_4
    List.singleton(Nat.4).filter(order_four_pred_mod_five).length = Nat.0
    Nat.5.range.filter(order_four_pred_mod_five).length =
        Nat.4.range.filter(order_four_pred_mod_five).length + Nat.0
    // 4.range = 3.range + [3]
    range_four_mod_five
    Nat.4.range = Nat.3.range + List.singleton(Nat.3)
    filter_add_length(Nat.3.range, List.singleton(Nat.3), order_four_pred_mod_five)
    Nat.4.range.filter(order_four_pred_mod_five).length =
        Nat.3.range.filter(order_four_pred_mod_five).length +
            List.singleton(Nat.3).filter(order_four_pred_mod_five).length
    singleton_order_four_len_3
    List.singleton(Nat.3).filter(order_four_pred_mod_five).length = Nat.1
    Nat.4.range.filter(order_four_pred_mod_five).length =
        Nat.3.range.filter(order_four_pred_mod_five).length + Nat.1
    // 3.range = 2.range + [2]
    range_three_mod_five
    Nat.3.range = Nat.2.range + List.singleton(Nat.2)
    filter_add_length(Nat.2.range, List.singleton(Nat.2), order_four_pred_mod_five)
    Nat.3.range.filter(order_four_pred_mod_five).length =
        Nat.2.range.filter(order_four_pred_mod_five).length +
            List.singleton(Nat.2).filter(order_four_pred_mod_five).length
    singleton_order_four_len_2
    List.singleton(Nat.2).filter(order_four_pred_mod_five).length = Nat.1
    Nat.3.range.filter(order_four_pred_mod_five).length =
        Nat.2.range.filter(order_four_pred_mod_five).length + Nat.1
    // 2.range = 1.range + [1]
    range_two_mod_five
    Nat.2.range = Nat.1.range + List.singleton(Nat.1)
    filter_add_length(Nat.1.range, List.singleton(Nat.1), order_four_pred_mod_five)
    Nat.2.range.filter(order_four_pred_mod_five).length =
        Nat.1.range.filter(order_four_pred_mod_five).length +
            List.singleton(Nat.1).filter(order_four_pred_mod_five).length
    singleton_order_four_len_1
    List.singleton(Nat.1).filter(order_four_pred_mod_five).length = Nat.0
    Nat.2.range.filter(order_four_pred_mod_five).length =
        Nat.1.range.filter(order_four_pred_mod_five).length + Nat.0
    // 1.range = 0.range + [0]
    range_one_mod_five
    Nat.1.range = Nat.0.range + List.singleton(Nat.0)
    filter_add_length(Nat.0.range, List.singleton(Nat.0), order_four_pred_mod_five)
    Nat.1.range.filter(order_four_pred_mod_five).length =
        Nat.0.range.filter(order_four_pred_mod_five).length +
            List.singleton(Nat.0).filter(order_four_pred_mod_five).length
    singleton_order_four_len_0
    List.singleton(Nat.0).filter(order_four_pred_mod_five).length = Nat.0
    Nat.1.range.filter(order_four_pred_mod_five).length =
        Nat.0.range.filter(order_four_pred_mod_five).length + Nat.0
    // 0.range = []
    range_zero_mod_five
    Nat.0.range = List.nil[Nat]
    List.nil[Nat].filter(order_four_pred_mod_five) = List.nil[Nat]
    Nat.0.range.filter(order_four_pred_mod_five) = List.nil[Nat]
    List.nil[Nat].filter(order_four_pred_mod_five).length = Nat.0
    Nat.0.range.filter(order_four_pred_mod_five).length = Nat.0
    // chain the counts back up
    Nat.1.range.filter(order_four_pred_mod_five).length = Nat.0 + Nat.0
    Nat.2.range.filter(order_four_pred_mod_five).length = Nat.0 + Nat.0 + Nat.0
    Nat.3.range.filter(order_four_pred_mod_five).length = Nat.0 + Nat.0 + Nat.0 + Nat.1
    Nat.4.range.filter(order_four_pred_mod_five).length = Nat.0 + Nat.0 + Nat.0 + Nat.1 + Nat.1
    Nat.5.range.filter(order_four_pred_mod_five).length =
        Nat.0 + Nat.0 + Nat.0 + Nat.1 + Nat.1 + Nat.0
    Nat.0 + Nat.0 + Nat.0 + Nat.1 + Nat.1 + Nat.0 = Nat.2
    Nat.5.range.filter(order_four_pred_mod_five).length = Nat.2
}

/// Filtering a singleton by a false predicate gives length `0`.
theorem singleton_filter_length_false(p: (Nat -> Bool), x: Nat) {
    not p(x) implies List.singleton(x).filter(p).length = Nat.0
} by {
    if not p(x) {
        filter_cons_of_false(x, List.nil[Nat], p)
        List.cons(x, List.nil[Nat]).filter(p) = List.nil[Nat].filter(p)
        List.singleton(x) = List.cons(x, List.nil[Nat])
        List.singleton(x).filter(p) = List.nil[Nat].filter(p)
        List.nil[Nat].filter(p) = List.nil[Nat]
        List.singleton(x).filter(p) = List.nil[Nat]
        List.singleton(x).filter(p).length = Nat.0
    }
}

/// Filtering a singleton by a true predicate gives length `1`.
theorem singleton_filter_length_true(p: (Nat -> Bool), x: Nat) {
    p(x) implies List.singleton(x).filter(p).length = Nat.1
} by {
    if p(x) {
        filter_cons_of_true(x, List.nil[Nat], p)
        List.cons(x, List.nil[Nat]).filter(p) = List.cons(x, List.nil[Nat].filter(p))
        List.singleton(x) = List.cons(x, List.nil[Nat])
        List.singleton(x).filter(p) = List.cons(x, List.nil[Nat].filter(p))
        List.nil[Nat].filter(p) = List.nil[Nat]
        List.singleton(x).filter(p) = List.singleton(x)
        List.singleton(x).length = Nat.1
        List.singleton(x).filter(p).length = Nat.1
    }
}

/// A predicate true exactly at `2` and `3` selects two elements of `5.range`.
theorem filter_five_count_two_three(p: (Nat -> Bool)) {
    not p(Nat.0) and not p(Nat.1) and p(Nat.2) and p(Nat.3) and not p(Nat.4)
        implies Nat.5.range.filter(p).length = Nat.2
} by {
    if not p(Nat.0) and not p(Nat.1) and p(Nat.2) and p(Nat.3) and not p(Nat.4) {
        Nat.5.range = Nat.4.range + List.singleton(Nat.4)
        filter_add_length(Nat.4.range, List.singleton(Nat.4), p)
        Nat.5.range.filter(p).length =
            Nat.4.range.filter(p).length + List.singleton(Nat.4).filter(p).length
        singleton_filter_length_false(p, Nat.4)
        List.singleton(Nat.4).filter(p).length = Nat.0
        Nat.5.range.filter(p).length = Nat.4.range.filter(p).length + Nat.0
        Nat.4.range = Nat.3.range + List.singleton(Nat.3)
        filter_add_length(Nat.3.range, List.singleton(Nat.3), p)
        Nat.4.range.filter(p).length =
            Nat.3.range.filter(p).length + List.singleton(Nat.3).filter(p).length
        singleton_filter_length_true(p, Nat.3)
        List.singleton(Nat.3).filter(p).length = Nat.1
        Nat.4.range.filter(p).length = Nat.3.range.filter(p).length + Nat.1
        Nat.3.range = Nat.2.range + List.singleton(Nat.2)
        filter_add_length(Nat.2.range, List.singleton(Nat.2), p)
        Nat.3.range.filter(p).length =
            Nat.2.range.filter(p).length + List.singleton(Nat.2).filter(p).length
        singleton_filter_length_true(p, Nat.2)
        List.singleton(Nat.2).filter(p).length = Nat.1
        Nat.3.range.filter(p).length = Nat.2.range.filter(p).length + Nat.1
        Nat.2.range = Nat.1.range + List.singleton(Nat.1)
        filter_add_length(Nat.1.range, List.singleton(Nat.1), p)
        Nat.2.range.filter(p).length =
            Nat.1.range.filter(p).length + List.singleton(Nat.1).filter(p).length
        singleton_filter_length_false(p, Nat.1)
        List.singleton(Nat.1).filter(p).length = Nat.0
        Nat.2.range.filter(p).length = Nat.1.range.filter(p).length + Nat.0
        Nat.1.range = Nat.0.range + List.singleton(Nat.0)
        filter_add_length(Nat.0.range, List.singleton(Nat.0), p)
        Nat.1.range.filter(p).length =
            Nat.0.range.filter(p).length + List.singleton(Nat.0).filter(p).length
        singleton_filter_length_false(p, Nat.0)
        List.singleton(Nat.0).filter(p).length = Nat.0
        Nat.1.range.filter(p).length = Nat.0.range.filter(p).length + Nat.0
        Nat.0.range = List.nil[Nat]
        List.nil[Nat].filter(p) = List.nil[Nat]
        Nat.0.range.filter(p) = List.nil[Nat]
        List.nil[Nat].filter(p).length = Nat.0
        Nat.0.range.filter(p).length = Nat.0
        Nat.1.range.filter(p).length = Nat.0 + Nat.0
        Nat.2.range.filter(p).length = Nat.0 + Nat.0 + Nat.0
        Nat.3.range.filter(p).length = Nat.0 + Nat.0 + Nat.0 + Nat.1
        Nat.4.range.filter(p).length = Nat.0 + Nat.0 + Nat.0 + Nat.1 + Nat.1
        Nat.5.range.filter(p).length = Nat.0 + Nat.0 + Nat.0 + Nat.1 + Nat.1 + Nat.0
        Nat.0 + Nat.0 + Nat.0 + Nat.1 + Nat.1 + Nat.0 = Nat.2
        Nat.5.range.filter(p).length = Nat.2
    }
}


/// A predicate true exactly at `4` selects one element of `5.range`.
theorem filter_five_count_four_only(p: (Nat -> Bool)) {
    not p(Nat.0) and not p(Nat.1) and not p(Nat.2) and not p(Nat.3) and p(Nat.4)
        implies Nat.5.range.filter(p).length = Nat.1
} by {
    if not p(Nat.0) and not p(Nat.1) and not p(Nat.2) and not p(Nat.3) and p(Nat.4) {
        Nat.5.range = Nat.4.range + List.singleton(Nat.4)
        filter_add_length(Nat.4.range, List.singleton(Nat.4), p)
        Nat.5.range.filter(p).length =
            Nat.4.range.filter(p).length + List.singleton(Nat.4).filter(p).length
        singleton_filter_length_true(p, Nat.4)
        List.singleton(Nat.4).filter(p).length = Nat.1
        Nat.5.range.filter(p).length = Nat.4.range.filter(p).length + Nat.1
        Nat.4.range = Nat.3.range + List.singleton(Nat.3)
        filter_add_length(Nat.3.range, List.singleton(Nat.3), p)
        Nat.4.range.filter(p).length =
            Nat.3.range.filter(p).length + List.singleton(Nat.3).filter(p).length
        singleton_filter_length_false(p, Nat.3)
        List.singleton(Nat.3).filter(p).length = Nat.0
        Nat.4.range.filter(p).length = Nat.3.range.filter(p).length + Nat.0
        Nat.3.range = Nat.2.range + List.singleton(Nat.2)
        filter_add_length(Nat.2.range, List.singleton(Nat.2), p)
        Nat.3.range.filter(p).length =
            Nat.2.range.filter(p).length + List.singleton(Nat.2).filter(p).length
        singleton_filter_length_false(p, Nat.2)
        List.singleton(Nat.2).filter(p).length = Nat.0
        Nat.3.range.filter(p).length = Nat.2.range.filter(p).length + Nat.0
        Nat.2.range = Nat.1.range + List.singleton(Nat.1)
        filter_add_length(Nat.1.range, List.singleton(Nat.1), p)
        Nat.2.range.filter(p).length =
            Nat.1.range.filter(p).length + List.singleton(Nat.1).filter(p).length
        singleton_filter_length_false(p, Nat.1)
        List.singleton(Nat.1).filter(p).length = Nat.0
        Nat.2.range.filter(p).length = Nat.1.range.filter(p).length + Nat.0
        Nat.1.range = Nat.0.range + List.singleton(Nat.0)
        filter_add_length(Nat.0.range, List.singleton(Nat.0), p)
        Nat.1.range.filter(p).length =
            Nat.0.range.filter(p).length + List.singleton(Nat.0).filter(p).length
        singleton_filter_length_false(p, Nat.0)
        List.singleton(Nat.0).filter(p).length = Nat.0
        Nat.1.range.filter(p).length = Nat.0.range.filter(p).length + Nat.0
        Nat.0.range = List.nil[Nat]
        List.nil[Nat].filter(p) = List.nil[Nat]
        Nat.0.range.filter(p) = List.nil[Nat]
        List.nil[Nat].filter(p).length = Nat.0
        Nat.0.range.filter(p).length = Nat.0
        Nat.1.range.filter(p).length = Nat.0 + Nat.0
        Nat.2.range.filter(p).length = Nat.0 + Nat.0 + Nat.0
        Nat.3.range.filter(p).length = Nat.0 + Nat.0 + Nat.0 + Nat.0
        Nat.4.range.filter(p).length = Nat.0 + Nat.0 + Nat.0 + Nat.0 + Nat.0
        Nat.5.range.filter(p).length = Nat.0 + Nat.0 + Nat.0 + Nat.0 + Nat.0 + Nat.1
        Nat.0 + Nat.0 + Nat.0 + Nat.0 + Nat.0 + Nat.1 = Nat.1
        Nat.5.range.filter(p).length = Nat.1
    }
}

/// A predicate true exactly at `1` selects one element of `5.range`.
theorem filter_five_count_one_only(p: (Nat -> Bool)) {
    not p(Nat.0) and p(Nat.1) and not p(Nat.2) and not p(Nat.3) and not p(Nat.4)
        implies Nat.5.range.filter(p).length = Nat.1
} by {
    if not p(Nat.0) and p(Nat.1) and not p(Nat.2) and not p(Nat.3) and not p(Nat.4) {
        Nat.5.range = Nat.4.range + List.singleton(Nat.4)
        filter_add_length(Nat.4.range, List.singleton(Nat.4), p)
        Nat.5.range.filter(p).length =
            Nat.4.range.filter(p).length + List.singleton(Nat.4).filter(p).length
        singleton_filter_length_false(p, Nat.4)
        List.singleton(Nat.4).filter(p).length = Nat.0
        Nat.5.range.filter(p).length = Nat.4.range.filter(p).length + Nat.0
        Nat.4.range = Nat.3.range + List.singleton(Nat.3)
        filter_add_length(Nat.3.range, List.singleton(Nat.3), p)
        Nat.4.range.filter(p).length =
            Nat.3.range.filter(p).length + List.singleton(Nat.3).filter(p).length
        singleton_filter_length_false(p, Nat.3)
        List.singleton(Nat.3).filter(p).length = Nat.0
        Nat.4.range.filter(p).length = Nat.3.range.filter(p).length + Nat.0
        Nat.3.range = Nat.2.range + List.singleton(Nat.2)
        filter_add_length(Nat.2.range, List.singleton(Nat.2), p)
        Nat.3.range.filter(p).length =
            Nat.2.range.filter(p).length + List.singleton(Nat.2).filter(p).length
        singleton_filter_length_false(p, Nat.2)
        List.singleton(Nat.2).filter(p).length = Nat.0
        Nat.3.range.filter(p).length = Nat.2.range.filter(p).length + Nat.0
        Nat.2.range = Nat.1.range + List.singleton(Nat.1)
        filter_add_length(Nat.1.range, List.singleton(Nat.1), p)
        Nat.2.range.filter(p).length =
            Nat.1.range.filter(p).length + List.singleton(Nat.1).filter(p).length
        singleton_filter_length_true(p, Nat.1)
        List.singleton(Nat.1).filter(p).length = Nat.1
        Nat.2.range.filter(p).length = Nat.1.range.filter(p).length + Nat.1
        Nat.1.range = Nat.0.range + List.singleton(Nat.0)
        filter_add_length(Nat.0.range, List.singleton(Nat.0), p)
        Nat.1.range.filter(p).length =
            Nat.0.range.filter(p).length + List.singleton(Nat.0).filter(p).length
        singleton_filter_length_false(p, Nat.0)
        List.singleton(Nat.0).filter(p).length = Nat.0
        Nat.1.range.filter(p).length = Nat.0.range.filter(p).length + Nat.0
        Nat.0.range = List.nil[Nat]
        List.nil[Nat].filter(p) = List.nil[Nat]
        Nat.0.range.filter(p) = List.nil[Nat]
        List.nil[Nat].filter(p).length = Nat.0
        Nat.0.range.filter(p).length = Nat.0
        Nat.1.range.filter(p).length = Nat.0 + Nat.0
        Nat.2.range.filter(p).length = Nat.0 + Nat.0 + Nat.1
        Nat.3.range.filter(p).length = Nat.0 + Nat.0 + Nat.1 + Nat.0
        Nat.4.range.filter(p).length = Nat.0 + Nat.0 + Nat.1 + Nat.0 + Nat.0
        Nat.5.range.filter(p).length = Nat.0 + Nat.0 + Nat.1 + Nat.0 + Nat.0 + Nat.0
        Nat.0 + Nat.0 + Nat.1 + Nat.0 + Nat.0 + Nat.0 = Nat.1
        Nat.5.range.filter(p).length = Nat.1
    }
}


/// The order-`2` residues modulo `5`.
define order_two_pred_mod_five(x: Nat) -> Bool {
    multiplicative_order_mod(x, Nat.5) = Nat.2
}

/// The order-`1` residues modulo `5`.
define order_one_pred_mod_five(x: Nat) -> Bool {
    multiplicative_order_mod(x, Nat.5) = Nat.1
}

theorem not_order_four_pred_0 {
    not order_four_pred_mod_five(Nat.0)
} by {
    order_zero_mod_five
    multiplicative_order_mod(Nat.0, Nat.5) = Nat.0
    if order_four_pred_mod_five(Nat.0) {
        multiplicative_order_mod(Nat.0, Nat.5) = Nat.4
        Nat.0 = Nat.4
        false
    }
}

theorem not_order_four_pred_1 {
    not order_four_pred_mod_five(Nat.1)
} by {
    order_one_mod_five
    multiplicative_order_mod(Nat.1, Nat.5) = Nat.1
    if order_four_pred_mod_five(Nat.1) {
        multiplicative_order_mod(Nat.1, Nat.5) = Nat.4
        Nat.1 = Nat.4
        false
    }
}

theorem not_order_four_pred_4 {
    not order_four_pred_mod_five(Nat.4)
} by {
    order_four_mod_five
    multiplicative_order_mod(Nat.4, Nat.5) = Nat.2
    if order_four_pred_mod_five(Nat.4) {
        multiplicative_order_mod(Nat.4, Nat.5) = Nat.4
        Nat.2 = Nat.4
        false
    }
}

theorem order_four_pred_2 {
    order_four_pred_mod_five(Nat.2)
} by {
    order_two_mod_five
    multiplicative_order_mod(Nat.2, Nat.5) = Nat.4
}

theorem order_four_pred_3 {
    order_four_pred_mod_five(Nat.3)
} by {
    order_three_mod_five
    multiplicative_order_mod(Nat.3, Nat.5) = Nat.4
}

theorem order_two_pred_4 {
    order_two_pred_mod_five(Nat.4)
} by {
    order_four_mod_five
    multiplicative_order_mod(Nat.4, Nat.5) = Nat.2
}

theorem not_order_two_pred_0 {
    not order_two_pred_mod_five(Nat.0)
} by {
    order_zero_mod_five
    multiplicative_order_mod(Nat.0, Nat.5) = Nat.0
    if order_two_pred_mod_five(Nat.0) {
        multiplicative_order_mod(Nat.0, Nat.5) = Nat.2
        Nat.0 = Nat.2
        false
    }
}

theorem not_order_two_pred_1 {
    not order_two_pred_mod_five(Nat.1)
} by {
    order_one_mod_five
    multiplicative_order_mod(Nat.1, Nat.5) = Nat.1
    if order_two_pred_mod_five(Nat.1) {
        multiplicative_order_mod(Nat.1, Nat.5) = Nat.2
        Nat.1 = Nat.2
        false
    }
}

theorem not_order_two_pred_2 {
    not order_two_pred_mod_five(Nat.2)
} by {
    order_two_mod_five
    multiplicative_order_mod(Nat.2, Nat.5) = Nat.4
    if order_two_pred_mod_five(Nat.2) {
        multiplicative_order_mod(Nat.2, Nat.5) = Nat.2
        Nat.4 = Nat.2
        false
    }
}

theorem not_order_two_pred_3 {
    not order_two_pred_mod_five(Nat.3)
} by {
    order_three_mod_five
    multiplicative_order_mod(Nat.3, Nat.5) = Nat.4
    if order_two_pred_mod_five(Nat.3) {
        multiplicative_order_mod(Nat.3, Nat.5) = Nat.2
        Nat.4 = Nat.2
        false
    }
}

theorem order_one_pred_1 {
    order_one_pred_mod_five(Nat.1)
} by {
    order_one_mod_five
    multiplicative_order_mod(Nat.1, Nat.5) = Nat.1
}

theorem not_order_one_pred_0 {
    not order_one_pred_mod_five(Nat.0)
} by {
    order_zero_mod_five
    multiplicative_order_mod(Nat.0, Nat.5) = Nat.0
    if order_one_pred_mod_five(Nat.0) {
        multiplicative_order_mod(Nat.0, Nat.5) = Nat.1
        Nat.0 = Nat.1
        false
    }
}

theorem not_order_one_pred_2 {
    not order_one_pred_mod_five(Nat.2)
} by {
    order_two_mod_five
    multiplicative_order_mod(Nat.2, Nat.5) = Nat.4
    if order_one_pred_mod_five(Nat.2) {
        multiplicative_order_mod(Nat.2, Nat.5) = Nat.1
        Nat.4 = Nat.1
        false
    }
}

theorem not_order_one_pred_3 {
    not order_one_pred_mod_five(Nat.3)
} by {
    order_three_mod_five
    multiplicative_order_mod(Nat.3, Nat.5) = Nat.4
    if order_one_pred_mod_five(Nat.3) {
        multiplicative_order_mod(Nat.3, Nat.5) = Nat.1
        Nat.4 = Nat.1
        false
    }
}

theorem not_order_one_pred_4 {
    not order_one_pred_mod_five(Nat.4)
} by {
    order_four_mod_five
    multiplicative_order_mod(Nat.4, Nat.5) = Nat.2
    if order_one_pred_mod_five(Nat.4) {
        multiplicative_order_mod(Nat.4, Nat.5) = Nat.1
        Nat.2 = Nat.1
        false
    }
}


/// Exactly `φ(4) = 2` elements have multiplicative order `4` modulo `5`.
theorem elements_of_order_four_count_mod_five {
    Nat.5.range.filter(order_four_pred_mod_five).length = Nat.4.totient
} by {
    not_order_four_pred_0
    not order_four_pred_mod_five(Nat.0)
    not_order_four_pred_1
    not order_four_pred_mod_five(Nat.1)
    order_four_pred_2
    order_four_pred_mod_five(Nat.2)
    order_four_pred_3
    order_four_pred_mod_five(Nat.3)
    not_order_four_pred_4
    not order_four_pred_mod_five(Nat.4)
    not order_four_pred_mod_five(Nat.0) and not order_four_pred_mod_five(Nat.1) and
        order_four_pred_mod_five(Nat.2) and order_four_pred_mod_five(Nat.3) and
        not order_four_pred_mod_five(Nat.4)
    filter_five_count_two_three(order_four_pred_mod_five)
    Nat.5.range.filter(order_four_pred_mod_five).length = Nat.2
    totient_four_mod_five
    Nat.4.totient = Nat.2
    Nat.5.range.filter(order_four_pred_mod_five).length = Nat.4.totient
}

/// Exactly `φ(2) = 1` element has multiplicative order `2` modulo `5`.
theorem elements_of_order_two_count_mod_five {
    Nat.5.range.filter(order_two_pred_mod_five).length = Nat.2.totient
} by {
    not_order_two_pred_0
    not order_two_pred_mod_five(Nat.0)
    not_order_two_pred_1
    not order_two_pred_mod_five(Nat.1)
    not_order_two_pred_2
    not order_two_pred_mod_five(Nat.2)
    not_order_two_pred_3
    not order_two_pred_mod_five(Nat.3)
    order_two_pred_4
    order_two_pred_mod_five(Nat.4)
    not order_two_pred_mod_five(Nat.0) and not order_two_pred_mod_five(Nat.1) and
        not order_two_pred_mod_five(Nat.2) and not order_two_pred_mod_five(Nat.3) and
        order_two_pred_mod_five(Nat.4)
    filter_five_count_four_only(order_two_pred_mod_five)
    Nat.5.range.filter(order_two_pred_mod_five).length = Nat.1
    totient_two_mod_five
    Nat.2.totient = Nat.1
    Nat.5.range.filter(order_two_pred_mod_five).length = Nat.2.totient
}

/// Exactly `φ(1) = 1` element has multiplicative order `1` modulo `5`.
theorem elements_of_order_one_count_mod_five {
    Nat.5.range.filter(order_one_pred_mod_five).length = Nat.1.totient
} by {
    not_order_one_pred_0
    not order_one_pred_mod_five(Nat.0)
    order_one_pred_1
    order_one_pred_mod_five(Nat.1)
    not_order_one_pred_2
    not order_one_pred_mod_five(Nat.2)
    not_order_one_pred_3
    not order_one_pred_mod_five(Nat.3)
    not_order_one_pred_4
    not order_one_pred_mod_five(Nat.4)
    not order_one_pred_mod_five(Nat.0) and order_one_pred_mod_five(Nat.1) and
        not order_one_pred_mod_five(Nat.2) and not order_one_pred_mod_five(Nat.3) and
        not order_one_pred_mod_five(Nat.4)
    filter_five_count_one_only(order_one_pred_mod_five)
    Nat.5.range.filter(order_one_pred_mod_five).length = Nat.1
    totient_one_mod_five
    Nat.1.totient = Nat.1
    Nat.5.range.filter(order_one_pred_mod_five).length = Nat.1.totient
}
