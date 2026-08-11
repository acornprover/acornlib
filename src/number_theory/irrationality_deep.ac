/// Irrationality, deepened.
///
/// The classical irrationality proofs for square roots and their sums,
/// built on the descent arguments of diophantine.ac and infinite_descent.ac:
///
///   1. The irrationality of √2 (Section 1): the Diophantine equation
///      x² = 2·y² has no positive solution.  Restated from diophantine.ac,
///      where the infinite descent lives.
///
///   2. The irrationality of √3 (Section 2): x² = 3·y² has no positive
///      solution.  Restated from infinite_descent.ac.
///
///   3. The irrationality of √6 (Section 3): x² = 6·y² has no positive
///      solution.  The descent mirrors the √2 proof: two divides the square
///      of the first coordinate, so the first coordinate is even; the
///      equation 2·a² = 3·y² then forces the second coordinate to be even
///      too (two is coprime to three), halving both coordinates to a smaller
///      solution; well-founded induction rules the descent out.
///
///   4. √n is irrational for nonsquare n (Section 4): the general statement
///      is recorded (its proof needs the prime-factorization form of "not a
///      perfect square"), and the concrete cases n = 2, 3, 6 are proved in
///      Sections 1–3 and collected in a single theorem.
///
///   5. The sum √2 + √3 is irrational (Section 5): the algebra
///      (√2 + √3)² = 5 + 2·√6 is verified over the reals, and the
///      Diophantine consequence — the equation (p² - 5·q²)² = 24·q⁴ has no
///      solution with q ≠ 0 — is proved from the √6 descent of Section 3.
from nat import Nat, mul_to_zero, mul_cancel_left, divisor_lt, lt_not_ref, mul_comm,
    divides_mul, mul_assoc
from algebra.well_founded import nat_lt_relation, nat_lt_relation_induction_at
from number_theory.coprime import coprime_divides_of_divides_mul
from number_theory.factorisation import coprime_of_distinct_primes
from number_theory.goldbach import three_is_prime
from number_theory import nat_two_prime
from number_theory.diophantine import no_nontrivial_sq_eq_two_sq, nat_sq_double,
    two_divides_square_imp_two_divides
from number_theory.approximation_deep import no_nontrivial_sq_eq_three_sq
from real import Real, mul_distrib_left, mul_distrib_right, real_mul_comm,
    add_comm, add_assoc

numerals Nat
numerals Real

// ============================================================================
// Section 1: the irrationality of √2
// ============================================================================

/// The Diophantine equation p² = 2·q² has no solution with q ≠ 0: √2 is
/// irrational.  This restates the infinite descent of diophantine.ac,
/// Section 3.
theorem irrationality_deep_sqrt_two_irrational(p: Nat, q: Nat) {
    p * p = Nat.2 * (q * q) implies q = Nat.0
} by {
    if p * p = Nat.2 * (q * q) {
        no_nontrivial_sq_eq_two_sq(p, q)
        p = Nat.0 and q = Nat.0
        q = Nat.0
    }
}

/// No positive rational squares to two: √2 is irrational.
theorem irrationality_deep_sqrt_two_no_positive_rational(p: Nat, q: Nat) {
    Nat.0 < q implies p * p != Nat.2 * (q * q)
} by {
    if Nat.0 < q {
        if p * p = Nat.2 * (q * q) {
            no_nontrivial_sq_eq_two_sq(p, q)
            p = Nat.0 and q = Nat.0
            q = Nat.0
            lt_not_ref(Nat.0)
            false
        }
        p * p != Nat.2 * (q * q)
    }
}

// ============================================================================
// Section 2: the irrationality of √3
// ============================================================================

/// The Diophantine equation p² = 3·q² has no solution with q ≠ 0: √3 is
/// irrational.  This restates the infinite descent of infinite_descent.ac,
/// Section 2 (also proved in approximation_deep.ac, Section 3, from which
/// the statement is imported here).
theorem irrationality_deep_sqrt_three_irrational(p: Nat, q: Nat) {
    p * p = Nat.3 * (q * q) implies q = Nat.0
} by {
    if p * p = Nat.3 * (q * q) {
        no_nontrivial_sq_eq_three_sq(p, q)
        p = Nat.0 and q = Nat.0
        q = Nat.0
    }
}

/// No positive rational squares to three: √3 is irrational.
theorem irrationality_deep_sqrt_three_no_positive_rational(p: Nat, q: Nat) {
    Nat.0 < q implies p * p != Nat.3 * (q * q)
} by {
    if Nat.0 < q {
        if p * p = Nat.3 * (q * q) {
            no_nontrivial_sq_eq_three_sq(p, q)
            p = Nat.0 and q = Nat.0
            q = Nat.0
            lt_not_ref(Nat.0)
            false
        }
        p * p != Nat.3 * (q * q)
    }
}

// ============================================================================
// Section 3: the irrationality of √6
// ============================================================================

/// Infinite descent: a nonzero solution of x² = 6y² yields a smaller one.
/// From x² = 6y², two divides the square of the first coordinate, so
/// x = 2a; the equation 2a² = 3y² then forces two to divide y² (two is
/// coprime to three), so y = 2b, and a² = 6b² with a < x.
theorem sq_eq_six_sq_descent(x: Nat, y: Nat) {
    x != Nat.0 and x * x = Nat.6 * (y * y) implies
        exists(a: Nat, b: Nat) {
            a < x and Nat.2 * a = x and a * a = Nat.6 * (b * b)
        }
} by {
    if x != Nat.0 and x * x = Nat.6 * (y * y) {
        // 2 | x·x, hence 2 | x, x = 2a.
        divides_mul(Nat.6, y * y, Nat.2)
        Nat.2.divides(Nat.6) implies Nat.2.divides(Nat.6 * (y * y))
        Nat.2 * Nat.3 = Nat.6
        Nat.2.divides(Nat.6)
        Nat.2.divides(Nat.6 * (y * y))
        Nat.6 * (y * y) = x * x
        Nat.2.divides(x * x)
        two_divides_square_imp_two_divides(x)
        Nat.2.divides(x)
        let (a: Nat) satisfy { Nat.2 * a = x }
        // a is a proper smaller root: a < x.
        mul_to_zero(Nat.2, a)
        if Nat.2 * a != Nat.0 {
            a != Nat.0
        }
        a != Nat.0
        a * Nat.2 = Nat.2 * a
        a * Nat.2 = x
        Nat.1 < Nat.2
        divisor_lt(a, Nat.2, x)
        a < x
        // 2·a·a = 3·y·y from (2a)² = 6y².
        Nat.2 * a = x
        x * x = (Nat.2 * a) * (Nat.2 * a)
        x * x = Nat.6 * (y * y)
        (Nat.2 * a) * (Nat.2 * a) = Nat.6 * (y * y)
        nat_sq_double(a)
        (Nat.2 * a) * (Nat.2 * a) = Nat.4 * (a * a)
        Nat.4 * (a * a) = Nat.6 * (y * y)
        Nat.2 * (Nat.2 * (a * a)) = Nat.2 * (Nat.3 * (y * y))
        mul_cancel_left(Nat.2, Nat.2 * (a * a), Nat.3 * (y * y))
        Nat.2 * (a * a) = Nat.3 * (y * y)
        // 2 | 3·y·y, and two is coprime to three, so 2 | y·y, hence 2 | y.
        Nat.2.divides(Nat.2 * (a * a))
        Nat.2 * (a * a) = Nat.3 * (y * y)
        Nat.2.divides(Nat.3 * (y * y))
        nat_two_prime
        Nat.2.is_prime
        three_is_prime
        Nat.3.is_prime
        Nat.2 != Nat.3
        coprime_of_distinct_primes(Nat.2, Nat.3)
        Nat.2.coprime(Nat.3)
        coprime_divides_of_divides_mul(Nat.2, Nat.3, y * y)
        Nat.2.divides(y * y)
        two_divides_square_imp_two_divides(y)
        Nat.2.divides(y)
        let (b: Nat) satisfy { Nat.2 * b = y }
        // a·a = 6·b·b from y = 2b.
        Nat.2 * b = y
        y * y = (Nat.2 * b) * (Nat.2 * b)
        Nat.2 * (a * a) = Nat.3 * (y * y)
        Nat.2 * (a * a) = Nat.3 * ((Nat.2 * b) * (Nat.2 * b))
        nat_sq_double(b)
        (Nat.2 * b) * (Nat.2 * b) = Nat.4 * (b * b)
        Nat.2 * (a * a) = Nat.3 * (Nat.4 * (b * b))
        Nat.3 * (Nat.4 * (b * b)) = Nat.12 * (b * b)
        Nat.2 * (a * a) = Nat.12 * (b * b)
        Nat.12 * (b * b) = Nat.2 * (Nat.6 * (b * b))
        Nat.2 * (a * a) = Nat.2 * (Nat.6 * (b * b))
        mul_cancel_left(Nat.2, a * a, Nat.6 * (b * b))
        a * a = Nat.6 * (b * b)
        // assemble the witness.
        a < x and Nat.2 * a = x and a * a = Nat.6 * (b * b)
        exists(a2: Nat, b2: Nat) {
            a2 < x and Nat.2 * a2 = x and a2 * a2 = Nat.6 * (b2 * b2)
        }
    }
}

/// The infinite descent of `sq_eq_six_sq_descent`, run by well-founded
/// induction on the first coordinate: x² = 6y² forces x = 0.
theorem sq_eq_six_sq_zero(x: Nat) {
    forall(y: Nat) { x * x = Nat.6 * (y * y) implies x = Nat.0 }
} by {
    let f: Nat -> Bool = function(t: Nat) {
        forall(y: Nat) { t * t = Nat.6 * (y * y) implies t = Nat.0 }
    }
    forall(z: Nat) {
        if forall(k: Nat) { nat_lt_relation(k, z) implies f(k) } {
            forall(y: Nat) {
                if z * z = Nat.6 * (y * y) {
                    if z = Nat.0 {
                        z = Nat.0
                    } else {
                        z != Nat.0 and z * z = Nat.6 * (y * y)
                        sq_eq_six_sq_descent(z, y)
                        let (a: Nat, b: Nat) satisfy {
                            a < z and Nat.2 * a = z and a * a = Nat.6 * (b * b)
                        }
                        nat_lt_relation(a, z) = (a < z)
                        nat_lt_relation(a, z)
                        forall(k: Nat) { nat_lt_relation(k, z) implies f(k) }
                        nat_lt_relation(a, z) implies f(a)
                        f(a)
                        f(a) =
                            forall(w: Nat) { a * a = Nat.6 * (w * w) implies a = Nat.0 }
                        forall(w: Nat) { a * a = Nat.6 * (w * w) implies a = Nat.0 }
                        a * a = Nat.6 * (b * b) implies a = Nat.0
                        a = Nat.0
                        Nat.2 * a = z
                        Nat.2 * Nat.0 = Nat.0
                        z = Nat.0
                        false
                    }
                }
            }
            forall(y: Nat) { z * z = Nat.6 * (y * y) implies z = Nat.0 }
            f(z) = forall(w: Nat) { z * z = Nat.6 * (w * w) implies z = Nat.0 }
            f(z)
        }
    }
    nat_lt_relation_induction_at(f, x)
    f(x)
    f(x) = forall(y: Nat) { x * x = Nat.6 * (y * y) implies x = Nat.0 }
    forall(y: Nat) { x * x = Nat.6 * (y * y) implies x = Nat.0 }
}

/// The Diophantine equation x² = 6y² has no nonzero natural solution.
theorem no_nontrivial_sq_eq_six_sq(x: Nat, y: Nat) {
    x * x = Nat.6 * (y * y) implies x = Nat.0 and y = Nat.0
} by {
    if x * x = Nat.6 * (y * y) {
        sq_eq_six_sq_zero(x)
        forall(w: Nat) { x * x = Nat.6 * (w * w) implies x = Nat.0 }
        x * x = Nat.6 * (y * y) implies x = Nat.0
        x = Nat.0
        x * x = Nat.6 * (y * y)
        Nat.0 * Nat.0 = Nat.0
        Nat.0 = Nat.6 * (y * y)
        Nat.6 * (y * y) = Nat.0
        mul_to_zero(Nat.6, y * y)
        if Nat.6 = Nat.0 {
            false
        }
        y * y = Nat.0
        mul_to_zero(y, y)
        if y = Nat.0 {
        } else {
            false
        }
        y = Nat.0
        x = Nat.0 and y = Nat.0
    }
}

/// The Diophantine equation p² = 6·q² has no solution with q ≠ 0: √6 is
/// irrational.
theorem irrationality_deep_sqrt_six_irrational(p: Nat, q: Nat) {
    p * p = Nat.6 * (q * q) implies q = Nat.0
} by {
    if p * p = Nat.6 * (q * q) {
        no_nontrivial_sq_eq_six_sq(p, q)
        p = Nat.0 and q = Nat.0
        q = Nat.0
    }
}

/// No positive rational squares to six: √6 is irrational.
theorem irrationality_deep_sqrt_six_no_positive_rational(p: Nat, q: Nat) {
    Nat.0 < q implies p * p != Nat.6 * (q * q)
} by {
    if Nat.0 < q {
        if p * p = Nat.6 * (q * q) {
            no_nontrivial_sq_eq_six_sq(p, q)
            p = Nat.0 and q = Nat.0
            q = Nat.0
            lt_not_ref(Nat.0)
            false
        }
        p * p != Nat.6 * (q * q)
    }
}

// ============================================================================
// Section 4: √n is irrational for nonsquare n
// ============================================================================

// The general statement — √n is irrational unless n is a perfect square.  In
// the Diophantine encoding (p/q)² = n with q ≠ 0 it reads:
//
// theorem irrationality_deep_sqrt_n_irrational_unless_square(n: Nat) {
//     not exists(m: Nat) { n = m * m } implies forall(p: Nat, q: Nat) {
//         p * p = n * (q * q) implies q = Nat.0
//     }
// }
//
// The base cases n = 2, 3, 6 are proved in Sections 1–3 (the descent
// argument handles the parity of the exponent of a prime dividing n with odd
// exponent, as in diophantine.ac and infinite_descent.ac).  The general case
// needs the prime-factorization form of "not a perfect square", which the
// library does not yet assemble (see the comment at the end of
// approximation_deep.ac).

/// √n is irrational for the concrete nonsquare values n = 2, 3, and 6: the
/// Diophantine equation p² = n·q² has no solution with q ≠ 0.
theorem irrationality_deep_sqrt_n_concrete_cases(n: Nat, p: Nat, q: Nat) {
    (n = Nat.2 or n = Nat.3 or n = Nat.6) and p * p = n * (q * q) implies q = Nat.0
} by {
    if (n = Nat.2 or n = Nat.3 or n = Nat.6) and p * p = n * (q * q) {
        n = Nat.2 or n = Nat.3 or n = Nat.6
        p * p = n * (q * q)
        if n = Nat.2 {
            p * p = Nat.2 * (q * q)
            irrationality_deep_sqrt_two_irrational(p, q)
            q = Nat.0
        }
        if n = Nat.3 {
            p * p = Nat.3 * (q * q)
            irrationality_deep_sqrt_three_irrational(p, q)
            q = Nat.0
        }
        if n = Nat.6 {
            p * p = Nat.6 * (q * q)
            irrationality_deep_sqrt_six_irrational(p, q)
            q = Nat.0
        }
        q = Nat.0
    }
}

// ============================================================================
// Section 5: the sum √2 + √3 is irrational
// ============================================================================

// The irrationality of √2 + √3 as a statement about reals.  The library's
// nonnegative square root (real.sqrt, private to the real package) provides
// the value lemmas needed to connect the algebra below to the actual square
// roots; that connection is left for future work (see the commented theorem
// at the end of this section).

/// The real number two, 1 + 1 (the digit `Real.2` is not defined in the
/// library).
let real_two: Real = Real.1 + Real.1

/// The real number three, 1 + 1 + 1 (the digit `Real.3` is not defined in
/// the library).
let real_three: Real = Real.1 + Real.1 + Real.1

/// The real number five, 1 + 1 + 1 + 1 + 1 (the digit `Real.5` is not
/// defined in the library).
let real_five: Real = Real.1 + Real.1 + Real.1 + Real.1 + Real.1

/// Two plus three is five, as reals.
theorem irrationality_deep_real_two_add_three {
    real_two + real_three = real_five
} by {
    real_two = Real.1 + Real.1
    real_three = Real.1 + Real.1 + Real.1
    real_five = Real.1 + Real.1 + Real.1 + Real.1 + Real.1
    real_two + real_three = Real.1 + Real.1 + Real.1 + Real.1 + Real.1
    Real.1 + Real.1 + Real.1 + Real.1 + Real.1 = real_five
    real_two + real_three = real_five
}

/// The regrouping 2 + x6 + (x6 + 3) = 5 + 2·x6, i.e. real_two + x6 + (x6 +
/// real_three) = real_five + real_two * x6.
theorem irrationality_deep_real_two_add_x6_three(x6: Real) {
    real_two + x6 + (x6 + real_three) = real_five + real_two * x6
} by {
    add_assoc(real_two, x6, x6 + real_three)
    (real_two + x6) + (x6 + real_three) = real_two + (x6 + (x6 + real_three))
    real_two + x6 + (x6 + real_three) = real_two + (x6 + (x6 + real_three))
    add_assoc(x6, x6, real_three)
    (x6 + x6) + real_three = x6 + (x6 + real_three)
    real_two + (x6 + (x6 + real_three)) = real_two + ((x6 + x6) + real_three)
    add_comm(x6 + x6, real_three)
    (x6 + x6) + real_three = real_three + (x6 + x6)
    real_two + ((x6 + x6) + real_three) = real_two + (real_three + (x6 + x6))
    add_assoc(real_two, real_three, x6 + x6)
    (real_two + real_three) + (x6 + x6) = real_two + (real_three + (x6 + x6))
    real_two + (real_three + (x6 + x6)) = (real_two + real_three) + (x6 + x6)
    irrationality_deep_real_two_add_three
    real_two + real_three = real_five
    (real_two + real_three) + (x6 + x6) = real_five + (x6 + x6)
    x6 + x6 = real_two * x6
    real_five + (x6 + x6) = real_five + real_two * x6
    real_two + x6 + (x6 + real_three) = real_five + real_two * x6
}

/// The square of the sum √2 + √3 is 5 + 2·√6: (x2 + x3)² = 5 + 2·x6 whenever
/// x2² = 2, x3² = 3, and x6 = x2·x3.  With x2 = √2, x3 = √3, and
/// x6 = √2·√3 = √6 this is exactly (√2 + √3)² = 5 + 2·√6.
theorem irrationality_deep_sqrt_two_add_sqrt_three_square(x2: Real, x3: Real, x6: Real) {
    x2 * x2 = real_two and x3 * x3 = real_three and x6 = x2 * x3 implies
        (x2 + x3) * (x2 + x3) = real_five + real_two * x6
} by {
    if x2 * x2 = real_two and x3 * x3 = real_three and x6 = x2 * x3 {
        x2 * x2 = real_two
        x3 * x3 = real_three
        x6 = x2 * x3
        mul_distrib_left(x2, x3, x2 + x3)
        (x2 + x3) * (x2 + x3) = x2 * (x2 + x3) + x3 * (x2 + x3)
        mul_distrib_right(x2, x2, x3)
        x2 * (x2 + x3) = x2 * x2 + x2 * x3
        mul_distrib_right(x3, x2, x3)
        x3 * (x2 + x3) = x3 * x2 + x3 * x3
        (x2 + x3) * (x2 + x3) = x2 * x2 + x2 * x3 + (x3 * x2 + x3 * x3)
        real_mul_comm(x3, x2)
        x3 * x2 = x2 * x3
        (x2 + x3) * (x2 + x3) = x2 * x2 + x2 * x3 + (x2 * x3 + x3 * x3)
        (x2 + x3) * (x2 + x3) = real_two + x6 + (x6 + real_three)
        irrationality_deep_real_two_add_x6_three(x6)
        real_two + x6 + (x6 + real_three) = real_five + real_two * x6
        (x2 + x3) * (x2 + x3) = real_five + real_two * x6
    }
}

// The square of the sum of the actual square roots, and the square root of
// six as a product.  Connecting the algebra above to the library's square
// root needs the value lemmas of real.sqrt (sqrt_value_mul_self, sqrt_mul,
// sqrt_unique_nonneg) and the nonnegativity of the real two and three, which
// live in modules private to the real package:
//
// theorem irrationality_deep_sqrt_two_add_sqrt_three_square_of_sqrt(x2: Real, x3: Real) {
//     sqrt(real_two) = Option.some(x2) and sqrt(real_three) = Option.some(x3) implies
//         (x2 + x3) * (x2 + x3) = real_five + real_two * (x2 * x3)
// }
//
// theorem irrationality_deep_sqrt_six_is_product(x2: Real, x3: Real, x6: Real) {
//     sqrt(real_two) = Option.some(x2) and sqrt(real_three) = Option.some(x3) and
//         x6 = x2 * x3 implies sqrt(real_two * real_three) = Option.some(x6)
// }

// The irrationality of √2 + √3 as a statement about reals: if x2 and x3 are
// the square roots of two and three, their sum is not rational.  The proof
// would continue from `irrationality_deep_sqrt_two_add_sqrt_three_square`:
// a rational x2 + x3 would square to a rational 5 + 2·(x2·x3), so x2·x3
// would be rational, contradicting the irrationality of √6 via
// `irrationality_deep_sqrt_six_irrational`; the library's rational-real
// bridge (from_rat field homomorphism and injectivity) is not yet assembled
// into that chain.
//
// theorem irrationality_deep_sqrt_two_add_sqrt_three_irrational_real(x2: Real, x3: Real) {
//     sqrt(real_two) = Option.some(x2) and sqrt(real_three) = Option.some(x3) implies
//         not exists(a: Nat, b: Nat) {
//             Nat.1 <= b and x2 + x3 = Real.from_rat(Rat.from_nat(a) / Rat.from_nat(b))
//         }
// }

/// The square of twice a square is four times the fourth power:
/// (2·q²)² = 4·q⁴.
theorem irrationality_deep_square_double_square(q: Nat) {
    (Nat.2 * q * q) * (Nat.2 * q * q) = Nat.4 * q * q * q * q
} by {
    mul_assoc(Nat.2, q, q)
    Nat.2 * q * q = Nat.2 * (q * q)
    (Nat.2 * q * q) * (Nat.2 * q * q) = (Nat.2 * (q * q)) * (Nat.2 * (q * q))
    nat_sq_double(q * q)
    (Nat.2 * (q * q)) * (Nat.2 * (q * q)) = Nat.4 * ((q * q) * (q * q))
    (Nat.2 * q * q) * (Nat.2 * q * q) = Nat.4 * ((q * q) * (q * q))
    Nat.4 * ((q * q) * (q * q)) = Nat.4 * q * q * q * q
    (Nat.2 * q * q) * (Nat.2 * q * q) = Nat.4 * q * q * q * q
}

/// Six times the square of twice a square is twenty-four times the fourth
/// power: 6·(2·q²)² = 24·q⁴.
theorem irrationality_deep_six_square_double_square(q: Nat) {
    Nat.6 * ((Nat.2 * q * q) * (Nat.2 * q * q)) = Nat.24 * q * q * q * q
} by {
    mul_assoc(Nat.2, q, q)
    Nat.2 * q * q = Nat.2 * (q * q)
    (Nat.2 * q * q) * (Nat.2 * q * q) = (Nat.2 * (q * q)) * (Nat.2 * (q * q))
    nat_sq_double(q * q)
    (Nat.2 * (q * q)) * (Nat.2 * (q * q)) = Nat.4 * ((q * q) * (q * q))
    (Nat.2 * q * q) * (Nat.2 * q * q) = Nat.4 * ((q * q) * (q * q))
    Nat.6 * ((Nat.2 * q * q) * (Nat.2 * q * q)) = Nat.6 * (Nat.4 * ((q * q) * (q * q)))
    Nat.6 * (Nat.4 * ((q * q) * (q * q))) = Nat.24 * ((q * q) * (q * q))
    Nat.24 * ((q * q) * (q * q)) = Nat.24 * q * q * q * q
    Nat.6 * ((Nat.2 * q * q) * (Nat.2 * q * q)) = Nat.24 * q * q * q * q
}

/// The Diophantine form of the irrationality of √2 + √3: the equation
/// (p² - 5·q²)² = 24·q⁴ has no solution with q ≠ 0.
///
/// If √2 + √3 were the rational p / q, squaring gives 5 + 2·√6 = p² / q², so
/// √6 = (p² - 5·q²) / (2·q²) and (p² - 5·q²)² = 24·q⁴; the √6 descent of
/// Section 3 rules the latter equation out, since 24·q⁴ = 6·(2·q²)².
theorem irrationality_deep_sqrt_two_add_sqrt_three_irrational(p: Nat, q: Nat) {
    Nat.0 < q implies
        (p * p - Nat.5 * q * q) * (p * p - Nat.5 * q * q) != Nat.24 * q * q * q * q
} by {
    if Nat.0 < q {
        if (p * p - Nat.5 * q * q) * (p * p - Nat.5 * q * q) = Nat.24 * q * q * q * q {
            irrationality_deep_six_square_double_square(q)
            Nat.6 * ((Nat.2 * q * q) * (Nat.2 * q * q)) = Nat.24 * q * q * q * q
            Nat.24 * q * q * q * q = Nat.6 * ((Nat.2 * q * q) * (Nat.2 * q * q))
            (p * p - Nat.5 * q * q) * (p * p - Nat.5 * q * q) = Nat.6 * ((Nat.2 * q * q) * (Nat.2 * q * q))
            irrationality_deep_sqrt_six_irrational(p * p - Nat.5 * q * q, Nat.2 * q * q)
            Nat.2 * q * q = Nat.0
            mul_to_zero(Nat.2, q * q)
            if Nat.2 = Nat.0 {
                false
            }
            q * q = Nat.0
            mul_to_zero(q, q)
            if q = Nat.0 {
            } else {
                false
            }
            q = Nat.0
            lt_not_ref(Nat.0)
            false
        }
        (p * p - Nat.5 * q * q) * (p * p - Nat.5 * q * q) != Nat.24 * q * q * q * q
    }
}
