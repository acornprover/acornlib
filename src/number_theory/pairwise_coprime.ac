from number_theory.coprime import Nat, coprime_one_right, coprime_mul
from list import List
from list import product
numerals Nat

/// True if every element of the tail is coprime with the given head.
define coprime_with_all(head: Nat, tail: List[Nat]) -> Bool {
    match tail {
        List.nil {
            true
        }
        List.cons(t_head, t_tail) {
            head.coprime(t_head) and coprime_with_all(head, t_tail)
        }
    }
}

/// True if every pair of distinct elements in the list is coprime.
define pairwise_coprime(list: List[Nat]) -> Bool {
    match list {
        List.nil {
            true
        }
        List.cons(head, tail) {
            coprime_with_all(head, tail) and pairwise_coprime(tail)
        }
    }
}

/// The empty list is pairwise coprime.
theorem pairwise_coprime_nil {
    pairwise_coprime(List.nil[Nat])
}

/// A singleton list is pairwise coprime (vacuously).
theorem pairwise_coprime_singleton(a: Nat) {
    pairwise_coprime(List.cons(a, List.nil[Nat]))
} by {
    coprime_with_all(a, List.nil[Nat])
}

/// Pairwise coprimality of a cons list implies the head is coprime with all
/// of the tail and the tail itself is pairwise coprime.
theorem pairwise_coprime_cons_imp(head: Nat, tail: List[Nat]) {
    pairwise_coprime(List.cons(head, tail)) implies
        coprime_with_all(head, tail) and pairwise_coprime(tail)
} by {
    if pairwise_coprime(List.cons(head, tail)) {
        coprime_with_all(head, tail) and pairwise_coprime(tail)
    }
}

/// Building up pairwise coprimality from the cons-pieces.
theorem pairwise_coprime_cons(head: Nat, tail: List[Nat]) {
    coprime_with_all(head, tail) and pairwise_coprime(tail) implies
        pairwise_coprime(List.cons(head, tail))
}

/// If a is coprime with every element of a list, then a is coprime with the
/// product of the list. The empty-list case uses `coprime_one_right`.
theorem coprime_with_all_imp_coprime_product(a: Nat, list: List[Nat]) {
    coprime_with_all(a, list) implies a.coprime(product[Nat](list))
} by {
    let f: List[Nat] -> Bool = function(l: List[Nat]) {
        coprime_with_all(a, l) implies a.coprime(product[Nat](l))
    }
    // Base case: empty list. product = 1, coprime with 1 always.
    coprime_one_right(a)
    a.coprime(Nat.1)
    product[Nat](List.nil[Nat]) = Nat.1
    f(List.nil[Nat])
    // Inductive step: list = cons(head, tail).
    forall(head: Nat, tail: List[Nat]) {
        if f(tail) {
            if coprime_with_all(a, List.cons(head, tail)) {
                a.coprime(head)
                coprime_with_all(a, tail)
                a.coprime(product[Nat](tail))
                coprime_mul(a, head, product[Nat](tail))
                a.coprime(head * product[Nat](tail))
                product[Nat](List.cons(head, tail)) = head * product[Nat](tail)
                a.coprime(product[Nat](List.cons(head, tail)))
            }
            f(List.cons(head, tail))
        }
    }
    f(list)
}
