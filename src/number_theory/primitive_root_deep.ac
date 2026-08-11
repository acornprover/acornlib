/// Primitive roots: deepening computations.
///
/// This file builds on `primitive_root.ac`, `primitive_root_applications.ac`,
/// `primitive_root_applications2.ac` and `orders.ac`.  It verifies that `2`
/// is a primitive root modulo `3`, restates the primitive root `3` modulo
/// `7` and the discrete logarithm, records the concrete counterexample that
/// the product of two primitive roots need not be a primitive root (modulo
/// `5`), and verifies the small case of the classical count of primitive
/// roots: `φ(p - 1)` for `p = 7`, where the primitive roots are exactly `3`
/// and `5` and `φ(6) = 2`.
///
/// Throughout, `g` is a primitive root modulo the prime `p`, formalized as
/// `p.is_prime and g.coprime(p) and multiplicative_order_mod(g, p) = p - Nat.1`
/// (the predicate `is_primitive_root_mod` of `primitive_root_applications2.ac`).

from number_theory.primitive_root_applications2 import Nat, is_primitive_root_mod,
    two_is_primitive_root_mod_five, primitive_root_discrete_log_exists,
    primitive_root_discrete_log_unique, primitive_root_discrete_log_well_defined,
    one_plus_one, two_plus_one, two_plus_two, two_plus_three, three_plus_three,
    four_plus_one, five_plus_one, one_plus_four, one_plus_five, lt_two_three,
    lt_zero_two, lt_zero_three, lt_zero_five, lt_zero_six, lt_two_four,
    lt_three_four, lt_five_six, lt_four_seven, lt_five_seven, lt_six_seven,
    pow_two_one, pow_two_two, pow_two_three, two_ne_one, three_ne_one,
    four_ne_one, five_ne_one, six_ne_one
from number_theory.orders import three_is_primitive_root_mod_seven,
    order_three_mod_seven, three_powers_mod_seven_table, three_coprime_mod_seven,
    mod_one_mod_seven, mod_two_mod_seven, mod_three_mod_seven, mod_four_mod_seven,
    mod_five_mod_seven, mod_six_mod_seven, not_congr_two_mod_seven,
    not_congr_three_mod_seven, not_congr_four_mod_seven, not_congr_five_mod_seven,
    not_congr_six_mod_seven, congr_thirty_six_mod_seven, lt_two_seven,
    three_is_primitive_root_mod_five, one_not_primitive_root_mod_five, one_ne_four
from number_theory.primitive_root_applications import mod_one_mod_five,
    lt_one_mod_five, lt_zero_one_mod_five
from number_theory.multiplicative_order import is_multiplicative_order_mod,
    multiplicative_order_mod, multiplicative_order_mod_is_order,
    multiplicative_order_mod_positive, multiplicative_order_pow_congr_one,
    multiplicative_order_mod_minimal
from number_theory.totient import coprime_below_prime, totient_pq
from number_theory.congruence import congr_mod_refl, congr_mod_symm,
    congr_mod_trans, congr_mod_mul, congr_mod_pow
from number_theory.coprime import coprime_one_left, coprime_unmod_imp,
    coprime_zero_left_imp_one
from number_theory.carmichael import two_is_prime, three_is_prime, lt_one_three
from number_theory.zsigmondy import seven_is_prime, lt_one_seven, lt_three_seven
from data.basic.logic import eq_true_intro, eq_false_intro
from nat import exp_one, exp_add, exp_mul, one_exp, small_mod, mod_of_decomp,
    add_imp_sub, add_one_right, add_assoc, lt_suc, lt_imp_lt_suc, lt_suc_right,
    lt_not_ref, not_lt_zero, trichotomy, lte_imp_not_lt, suc_sub_one,
    nat_mul_1_2, nat_mul_1_3, nat_mul_1_5, nat_mul_1_7, nat_mul_2_3,
    nat_mul_2_5, nat_mul_2_7, nat_mul_3_2, nat_mul_3_7, nat_mul_4_4,
    nat_mul_4_5, nat_mul_4_7, nat_mul_5_5, nat_mul_6_6, nat_add_7_2
numerals Nat

// ---------------------------------------------------------------------------
// Small arithmetic facts.
// ---------------------------------------------------------------------------

/// `3 + 1 = 4`.
theorem three_plus_one {
    Nat.3 + Nat.1 = Nat.4
} by {
    add_one_right(Nat.3)
    Nat.3 + Nat.1 = Nat.3.suc
    Nat.3.suc = Nat.4
    Nat.3 + Nat.1 = Nat.4
}

/// `7 + 3 = 10`.
theorem nat_add_7_3 {
    Nat.7 + Nat.3 = Nat.10
} by {
    add_assoc(Nat.7, Nat.2, Nat.1)
    Nat.7 + Nat.2 + Nat.1 = Nat.7 + (Nat.2 + Nat.1)
    two_plus_one
    Nat.2 + Nat.1 = Nat.3
    Nat.7 + Nat.3 = Nat.7 + Nat.2 + Nat.1
    nat_add_7_2
    Nat.7 + Nat.2 = Nat.9
    Nat.7 + Nat.3 = Nat.9 + Nat.1
    add_one_right(Nat.9)
    Nat.9 + Nat.1 = Nat.9.suc
    Nat.9.suc = Nat.10
    Nat.7 + Nat.3 = Nat.10
}

/// `7 + 1 = 8`.
theorem nat_add_7_1 {
    Nat.7 + Nat.1 = Nat.8
} by {
    add_one_right(Nat.7)
    Nat.7 + Nat.1 = Nat.7.suc
    Nat.7.suc = Nat.8
    Nat.7 + Nat.1 = Nat.8
}

/// `14 + 2 = 16`.
theorem nat_add_14_2 {
    Nat.14 + Nat.2 = Nat.16
} by {
    add_assoc(Nat.14, Nat.1, Nat.1)
    Nat.14 + Nat.1 + Nat.1 = Nat.14 + (Nat.1 + Nat.1)
    one_plus_one
    Nat.1 + Nat.1 = Nat.2
    Nat.14 + Nat.2 = Nat.14 + Nat.1 + Nat.1
    add_one_right(Nat.14)
    Nat.14 + Nat.1 = Nat.14.suc
    Nat.14.suc = Nat.15
    Nat.14 + Nat.1 = Nat.15
    add_one_right(Nat.15)
    Nat.15 + Nat.1 = Nat.15.suc
    Nat.15.suc = Nat.16
    Nat.15 + Nat.1 = Nat.16
    Nat.14 + Nat.2 = Nat.16
}

/// `14 + 6 = 20`.
theorem nat_add_14_6 {
    Nat.14 + Nat.6 = Nat.20
} by {
    add_assoc(Nat.14, Nat.5, Nat.1)
    Nat.14 + Nat.5 + Nat.1 = Nat.14 + (Nat.5 + Nat.1)
    one_plus_five
    Nat.1 + Nat.5 = Nat.6
    Nat.14 + Nat.6 = Nat.14 + Nat.5 + Nat.1
    add_assoc(Nat.14, Nat.4, Nat.1)
    Nat.14 + Nat.4 + Nat.1 = Nat.14 + (Nat.4 + Nat.1)
    one_plus_four
    Nat.1 + Nat.4 = Nat.5
    Nat.14 + Nat.5 = Nat.14 + Nat.4 + Nat.1
    add_assoc(Nat.14, Nat.3, Nat.1)
    Nat.14 + Nat.3 + Nat.1 = Nat.14 + (Nat.3 + Nat.1)
    three_plus_one
    Nat.3 + Nat.1 = Nat.4
    Nat.14 + Nat.4 = Nat.14 + Nat.3 + Nat.1
    add_assoc(Nat.14, Nat.2, Nat.1)
    Nat.14 + Nat.2 + Nat.1 = Nat.14 + (Nat.2 + Nat.1)
    two_plus_one
    Nat.2 + Nat.1 = Nat.3
    Nat.14 + Nat.3 = Nat.14 + Nat.2 + Nat.1
    nat_add_14_2
    Nat.14 + Nat.2 = Nat.16
    Nat.14 + Nat.3 = Nat.16 + Nat.1
    add_one_right(Nat.16)
    Nat.16 + Nat.1 = Nat.16.suc
    Nat.16.suc = Nat.17
    Nat.14 + Nat.3 = Nat.17
    Nat.14 + Nat.4 = Nat.17 + Nat.1
    add_one_right(Nat.17)
    Nat.17 + Nat.1 = Nat.17.suc
    Nat.17.suc = Nat.18
    Nat.14 + Nat.4 = Nat.18
    Nat.14 + Nat.5 = Nat.18 + Nat.1
    add_one_right(Nat.18)
    Nat.18 + Nat.1 = Nat.18.suc
    Nat.18.suc = Nat.19
    Nat.14 + Nat.5 = Nat.19
    Nat.14 + Nat.6 = Nat.19 + Nat.1
    add_one_right(Nat.19)
    Nat.19 + Nat.1 = Nat.19.suc
    Nat.19.suc = Nat.20
    Nat.14 + Nat.6 = Nat.20
}

/// `21 + 4 = 25`.
theorem nat_add_21_4 {
    Nat.21 + Nat.4 = Nat.25
} by {
    add_assoc(Nat.21, Nat.3, Nat.1)
    Nat.21 + Nat.3 + Nat.1 = Nat.21 + (Nat.3 + Nat.1)
    three_plus_one
    Nat.3 + Nat.1 = Nat.4
    Nat.21 + Nat.4 = Nat.21 + Nat.3 + Nat.1
    add_assoc(Nat.21, Nat.2, Nat.1)
    Nat.21 + Nat.2 + Nat.1 = Nat.21 + (Nat.2 + Nat.1)
    two_plus_one
    Nat.2 + Nat.1 = Nat.3
    Nat.21 + Nat.3 = Nat.21 + Nat.2 + Nat.1
    add_assoc(Nat.21, Nat.1, Nat.1)
    Nat.21 + Nat.1 + Nat.1 = Nat.21 + (Nat.1 + Nat.1)
    one_plus_one
    Nat.1 + Nat.1 = Nat.2
    Nat.21 + Nat.2 = Nat.21 + Nat.1 + Nat.1
    add_one_right(Nat.21)
    Nat.21 + Nat.1 = Nat.21.suc
    Nat.21.suc = Nat.22
    Nat.21 + Nat.1 = Nat.22
    add_one_right(Nat.22)
    Nat.22 + Nat.1 = Nat.22.suc
    Nat.22.suc = Nat.23
    Nat.22 + Nat.1 = Nat.23
    Nat.21 + Nat.2 = Nat.23
    Nat.21 + Nat.3 = Nat.23 + Nat.1
    add_one_right(Nat.23)
    Nat.23 + Nat.1 = Nat.23.suc
    Nat.23.suc = Nat.24
    Nat.21 + Nat.3 = Nat.24
    Nat.21 + Nat.4 = Nat.24 + Nat.1
    add_one_right(Nat.24)
    Nat.24 + Nat.1 = Nat.24.suc
    Nat.24.suc = Nat.25
    Nat.21 + Nat.4 = Nat.25
}

// ---------------------------------------------------------------------------
// The primitive root 2 modulo 3.
//
// The powers of `2` modulo `3` are `2^1 ≡ 2` and `2^2 = 4 ≡ 1`, so the
// multiplicative order of `2` modulo `3` is `2`, which is the full order
// `φ(3) = 3 - 1`.
// ---------------------------------------------------------------------------

/// `1.mod(3) = 1`.
theorem mod_one_mod_three {
    Nat.1.mod(Nat.3) = Nat.1
} by {
    lt_one_three
    Nat.1 < Nat.3
    small_mod(Nat.1, Nat.3)
}

/// `2.mod(3) = 2`.
theorem mod_two_mod_three {
    Nat.2.mod(Nat.3) = Nat.2
} by {
    lt_two_three
    Nat.2 < Nat.3
    small_mod(Nat.2, Nat.3)
}

/// `4 ≡ 1 (mod 3)`, since `4 = 1 · 3 + 1`.
theorem congr_four_mod_three {
    Nat.4.congr_mod(Nat.1, Nat.3)
} by {
    nat_mul_1_3
    Nat.1 * Nat.3 = Nat.3
    three_plus_one
    Nat.3 + Nat.1 = Nat.4
    lt_one_three
    Nat.1 < Nat.3
    mod_of_decomp(Nat.1, Nat.1, Nat.3)
    (Nat.1 * Nat.3 + Nat.1).mod(Nat.3) = Nat.1
    Nat.4.mod(Nat.3) = Nat.1
    mod_one_mod_three
    Nat.1.mod(Nat.3) = Nat.1
    Nat.4.mod(Nat.3) = Nat.1.mod(Nat.3)
    Nat.4.congr_mod(Nat.1, Nat.3)
}

/// `2^2 = 4 ≡ 1 (mod 3)`.
theorem congr_two_pow_two_mod_three {
    Nat.2.pow(Nat.2).congr_mod(Nat.1, Nat.3)
} by {
    pow_two_two
    Nat.2.pow(Nat.2) = Nat.4
    congr_four_mod_three
    Nat.4.congr_mod(Nat.1, Nat.3)
    Nat.2.pow(Nat.2).congr_mod(Nat.1, Nat.3)
}

/// `2 ≢ 1 (mod 3)`.
theorem not_congr_two_mod_three {
    not Nat.2.congr_mod(Nat.1, Nat.3)
} by {
    mod_two_mod_three
    Nat.2.mod(Nat.3) = Nat.2
    mod_one_mod_three
    Nat.1.mod(Nat.3) = Nat.1
    if Nat.2.congr_mod(Nat.1, Nat.3) {
        Nat.2.mod(Nat.3) = Nat.1.mod(Nat.3)
        Nat.2 = Nat.1
        two_ne_one
        Nat.2 != Nat.1
        false
    }
}

/// `2^1 = 2 ≢ 1 (mod 3)`.
theorem not_congr_two_pow_one_mod_three {
    not Nat.2.pow(Nat.1).congr_mod(Nat.1, Nat.3)
} by {
    exp_one(Nat.2)
    Nat.2.pow(Nat.1) = Nat.2
    not_congr_two_mod_three
    if Nat.2.pow(Nat.1).congr_mod(Nat.1, Nat.3) {
        Nat.2.congr_mod(Nat.1, Nat.3)
        false
    }
}

/// The powers of `2` modulo `3`: `2^1 ≡ 2` and `2^2 ≡ 1`.
theorem two_powers_mod_three_table {
    Nat.2.pow(Nat.1).congr_mod(Nat.2, Nat.3) and
        Nat.2.pow(Nat.2).congr_mod(Nat.1, Nat.3)
} by {
    exp_one(Nat.2)
    Nat.2.pow(Nat.1) = Nat.2
    congr_mod_refl(Nat.2, Nat.3)
    Nat.2.congr_mod(Nat.2, Nat.3)
    Nat.2.pow(Nat.1).congr_mod(Nat.2, Nat.3)
    congr_two_pow_two_mod_three
    Nat.2.pow(Nat.2).congr_mod(Nat.1, Nat.3)
    Nat.2.pow(Nat.1).congr_mod(Nat.2, Nat.3) and
        Nat.2.pow(Nat.2).congr_mod(Nat.1, Nat.3)
}

/// `3 - 1 = 2`.
theorem sub_three_one {
    Nat.3 - Nat.1 = Nat.2
} by {
    two_plus_one
    Nat.2 + Nat.1 = Nat.3
    add_imp_sub(Nat.2, Nat.1, Nat.3)
    Nat.3 - Nat.1 = Nat.2
}

/// `2` is coprime to `3`.
theorem two_coprime_mod_three {
    Nat.2.coprime(Nat.3)
} by {
    three_is_prime
    Nat.3.is_prime
    Nat.1 <= Nat.2
    lt_two_three
    Nat.2 < Nat.3
    coprime_below_prime(Nat.3, Nat.2)
    Nat.2.coprime(Nat.3)
}

/// `2` has multiplicative order `2` modulo `3`: `2^2 ≡ 1` but `2^1 ≢ 1`.
theorem order_two_mod_three {
    multiplicative_order_mod(Nat.2, Nat.3) = Nat.2
} by {
    Nat.3 != Nat.0
    two_coprime_mod_three
    Nat.2.coprime(Nat.3)
    lt_zero_two
    Nat.0 < Nat.2
    congr_two_pow_two_mod_three
    Nat.2.pow(Nat.2).congr_mod(Nat.1, Nat.3)
    multiplicative_order_mod_minimal(Nat.2, Nat.3, Nat.2)
    multiplicative_order_mod(Nat.2, Nat.3) <= Nat.2
    multiplicative_order_mod_positive(Nat.2, Nat.3)
    Nat.0 < multiplicative_order_mod(Nat.2, Nat.3)
    if multiplicative_order_mod(Nat.2, Nat.3) = Nat.1 {
        multiplicative_order_mod_is_order(Nat.2, Nat.3)
        is_multiplicative_order_mod(Nat.2, Nat.3, multiplicative_order_mod(Nat.2, Nat.3))
        is_multiplicative_order_mod(Nat.2, Nat.3, Nat.1)
        multiplicative_order_pow_congr_one(Nat.2, Nat.3, Nat.1)
        Nat.2.pow(Nat.1).congr_mod(Nat.1, Nat.3)
        not_congr_two_pow_one_mod_three
        false
    }
    trichotomy(multiplicative_order_mod(Nat.2, Nat.3), Nat.2)
    if multiplicative_order_mod(Nat.2, Nat.3) < Nat.2 {
        lt_suc_right(multiplicative_order_mod(Nat.2, Nat.3), Nat.1)
        if multiplicative_order_mod(Nat.2, Nat.3) = Nat.1 {
            false
        }
        multiplicative_order_mod(Nat.2, Nat.3) < Nat.1
        lt_suc_right(multiplicative_order_mod(Nat.2, Nat.3), Nat.0)
        if multiplicative_order_mod(Nat.2, Nat.3) = Nat.0 {
            Nat.0 < multiplicative_order_mod(Nat.2, Nat.3)
            Nat.0 < Nat.0
            lt_not_ref(Nat.0)
            false
        }
        multiplicative_order_mod(Nat.2, Nat.3) < Nat.0
        not_lt_zero(multiplicative_order_mod(Nat.2, Nat.3))
        false
    }
    if Nat.2 < multiplicative_order_mod(Nat.2, Nat.3) {
        lte_imp_not_lt(multiplicative_order_mod(Nat.2, Nat.3), Nat.2)
        not Nat.2 < multiplicative_order_mod(Nat.2, Nat.3)
        false
    }
    multiplicative_order_mod(Nat.2, Nat.3) = Nat.2
}

/// `2` is a primitive root modulo `3`: it is coprime to `3` and its order
/// `2` is the full order `φ(3) = 3 - 1`.
theorem two_is_primitive_root_mod_three {
    is_primitive_root_mod(Nat.2, Nat.3)
} by {
    three_is_prime
    Nat.3.is_prime
    two_coprime_mod_three
    Nat.2.coprime(Nat.3)
    order_two_mod_three
    multiplicative_order_mod(Nat.2, Nat.3) = Nat.2
    sub_three_one
    Nat.3 - Nat.1 = Nat.2
    multiplicative_order_mod(Nat.2, Nat.3) = Nat.3 - Nat.1
    Nat.3.is_prime and Nat.2.coprime(Nat.3) and
        multiplicative_order_mod(Nat.2, Nat.3) = Nat.3 - Nat.1
    is_primitive_root_mod(Nat.2, Nat.3) =
        (Nat.3.is_prime and Nat.2.coprime(Nat.3) and
            multiplicative_order_mod(Nat.2, Nat.3) = Nat.3 - Nat.1)
    is_primitive_root_mod(Nat.2, Nat.3)
}

// ---------------------------------------------------------------------------
// The primitive root 3 modulo 7 (restated from orders.ac).
// ---------------------------------------------------------------------------

/// `3` has multiplicative order `6` modulo `7` (restated from `orders.ac`).
theorem order_three_mod_seven_restated {
    multiplicative_order_mod(Nat.3, Nat.7) = Nat.6
} by {
    order_three_mod_seven
    multiplicative_order_mod(Nat.3, Nat.7) = Nat.6
}

/// The powers of `3` modulo `7`: `3^1 ≡ 3`, `3^2 ≡ 2`, `3^3 ≡ 6`, `3^4 ≡ 4`,
/// `3^5 ≡ 5`, and `3^6 ≡ 1` — all six nonzero residues (restated from
/// `orders.ac`).
theorem three_powers_mod_seven_table_restated {
    Nat.3.pow(Nat.1).congr_mod(Nat.3, Nat.7) and
        Nat.3.pow(Nat.2).congr_mod(Nat.2, Nat.7) and
        Nat.3.pow(Nat.3).congr_mod(Nat.6, Nat.7) and
        Nat.3.pow(Nat.4).congr_mod(Nat.4, Nat.7) and
        Nat.3.pow(Nat.5).congr_mod(Nat.5, Nat.7) and
        Nat.3.pow(Nat.6).congr_mod(Nat.1, Nat.7)
} by {
    three_powers_mod_seven_table
    Nat.3.pow(Nat.1).congr_mod(Nat.3, Nat.7) and
        Nat.3.pow(Nat.2).congr_mod(Nat.2, Nat.7) and
        Nat.3.pow(Nat.3).congr_mod(Nat.6, Nat.7) and
        Nat.3.pow(Nat.4).congr_mod(Nat.4, Nat.7) and
        Nat.3.pow(Nat.5).congr_mod(Nat.5, Nat.7) and
        Nat.3.pow(Nat.6).congr_mod(Nat.1, Nat.7)
}

/// `3` is a primitive root modulo `7` (restated from `orders.ac`).
theorem three_is_primitive_root_mod_seven_restated {
    is_primitive_root_mod(Nat.3, Nat.7)
} by {
    three_is_primitive_root_mod_seven
    is_primitive_root_mod(Nat.3, Nat.7)
}

// ---------------------------------------------------------------------------
// The discrete logarithm (restated from primitive_root_applications2.ac).
//
// For a primitive root `g` modulo the prime `p`, every residue `a` coprime to
// `p` has a unique exponent below `p - 1` whose power is congruent to it:
// the discrete logarithm of `a` to the base `g` is well defined.
// ---------------------------------------------------------------------------

/// Existence of the discrete logarithm: for a primitive root `g` modulo the
/// prime `p`, every residue `a` coprime to `p` is congruent to `g^e` for some
/// exponent `e` below `p - 1` (restated from `primitive_root_applications2.ac`).
theorem primitive_root_discrete_log_exists_restated(p: Nat, g: Nat, a: Nat) {
    p.is_prime and g.coprime(p) and multiplicative_order_mod(g, p) = p - Nat.1
        and a.coprime(p)
        implies exists(e: Nat) { e < p - Nat.1 and g.pow(e).congr_mod(a, p) }
} by {
    if p.is_prime and g.coprime(p) and multiplicative_order_mod(g, p) = p - Nat.1
        and a.coprime(p) {
        primitive_root_discrete_log_exists(p, g, a)
        exists(e: Nat) { e < p - Nat.1 and g.pow(e).congr_mod(a, p) }
    }
}

/// Uniqueness of the discrete logarithm: a residue `a` coprime to `p` has at
/// most one exponent below `p - 1` whose power is congruent to it (restated
/// from `primitive_root_applications2.ac`).
theorem primitive_root_discrete_log_unique_restated(
    p: Nat, g: Nat, a: Nat, e: Nat, f: Nat
) {
    p.is_prime and g.coprime(p) and multiplicative_order_mod(g, p) = p - Nat.1
        and a.coprime(p) and e < p - Nat.1 and f < p - Nat.1
        and g.pow(e).congr_mod(a, p) and g.pow(f).congr_mod(a, p)
        implies e = f
} by {
    if p.is_prime and g.coprime(p) and multiplicative_order_mod(g, p) = p - Nat.1
        and a.coprime(p) and e < p - Nat.1 and f < p - Nat.1
        and g.pow(e).congr_mod(a, p) and g.pow(f).congr_mod(a, p) {
        primitive_root_discrete_log_unique(p, g, a, e, f)
        e = f
    }
}

/// The discrete logarithm is well defined: for a primitive root `g` modulo
/// the prime `p`, every nonzero residue class `a` (coprime to `p`) has a
/// unique exponent below `p - 1` whose power is congruent to it (restated
/// from `primitive_root_applications2.ac`).
theorem primitive_root_discrete_log_well_defined_restated(p: Nat, g: Nat, a: Nat) {
    p.is_prime and g.coprime(p) and multiplicative_order_mod(g, p) = p - Nat.1
        and a.coprime(p)
        implies exists(e: Nat) { e < p - Nat.1 and g.pow(e).congr_mod(a, p) and
            forall(f: Nat) { f < p - Nat.1 and g.pow(f).congr_mod(a, p) implies f = e } }
} by {
    if p.is_prime and g.coprime(p) and multiplicative_order_mod(g, p) = p - Nat.1
        and a.coprime(p) {
        primitive_root_discrete_log_well_defined(p, g, a)
        exists(e: Nat) { e < p - Nat.1 and g.pow(e).congr_mod(a, p) and
            forall(f: Nat) { f < p - Nat.1 and g.pow(f).congr_mod(a, p) implies f = e } }
    }
}

// ---------------------------------------------------------------------------
// The product of two primitive roots need not be a primitive root.
//
// Modulo `5` the primitive roots are `2` and `3`, but their product
// `2 · 3 = 6` is congruent to `1` modulo `5`, and `1` (hence `6`) is not a
// primitive root modulo `5`.
// ---------------------------------------------------------------------------

/// `6 ≡ 1 (mod 5)`, since `6 = 1 · 5 + 1`.
theorem congr_six_mod_five {
    Nat.6.congr_mod(Nat.1, Nat.5)
} by {
    nat_mul_1_5
    Nat.1 * Nat.5 = Nat.5
    five_plus_one
    Nat.5 + Nat.1 = Nat.6
    lt_one_mod_five
    Nat.1 < Nat.5
    mod_of_decomp(Nat.1, Nat.1, Nat.5)
    (Nat.1 * Nat.5 + Nat.1).mod(Nat.5) = Nat.1
    Nat.6.mod(Nat.5) = Nat.1
    mod_one_mod_five
    Nat.1.mod(Nat.5) = Nat.1
    Nat.6.mod(Nat.5) = Nat.1.mod(Nat.5)
    Nat.6.congr_mod(Nat.1, Nat.5)
}

/// `6` is coprime to `5`: its reduction `6 mod 5 = 1` is coprime to `5`.
theorem six_coprime_mod_five {
    Nat.6.coprime(Nat.5)
} by {
    congr_six_mod_five
    Nat.6.mod(Nat.5) = Nat.1.mod(Nat.5)
    mod_one_mod_five
    Nat.1.mod(Nat.5) = Nat.1
    Nat.6.mod(Nat.5) = Nat.1
    coprime_one_left(Nat.5)
    Nat.1.coprime(Nat.5)
    Nat.6.mod(Nat.5).coprime(Nat.5)
    coprime_unmod_imp(Nat.6, Nat.5)
    Nat.6.coprime(Nat.5)
}

/// `6` has multiplicative order `1` modulo `5`, since `6 ≡ 1 (mod 5)`.
theorem order_six_mod_five {
    multiplicative_order_mod(Nat.6, Nat.5) = Nat.1
} by {
    Nat.5 != Nat.0
    six_coprime_mod_five
    Nat.6.coprime(Nat.5)
    lt_zero_one_mod_five
    Nat.0 < Nat.1
    congr_six_mod_five
    Nat.6.congr_mod(Nat.1, Nat.5)
    exp_one(Nat.6)
    Nat.6.pow(Nat.1) = Nat.6
    Nat.6.pow(Nat.1).congr_mod(Nat.1, Nat.5)
    multiplicative_order_mod_minimal(Nat.6, Nat.5, Nat.1)
    multiplicative_order_mod(Nat.6, Nat.5) <= Nat.1
    multiplicative_order_mod_positive(Nat.6, Nat.5)
    Nat.0 < multiplicative_order_mod(Nat.6, Nat.5)
    trichotomy(multiplicative_order_mod(Nat.6, Nat.5), Nat.1)
    if multiplicative_order_mod(Nat.6, Nat.5) < Nat.1 {
        lt_suc_right(multiplicative_order_mod(Nat.6, Nat.5), Nat.0)
        if multiplicative_order_mod(Nat.6, Nat.5) = Nat.0 {
            Nat.0 < multiplicative_order_mod(Nat.6, Nat.5)
            Nat.0 < Nat.0
            lt_not_ref(Nat.0)
            false
        }
        multiplicative_order_mod(Nat.6, Nat.5) < Nat.0
        not_lt_zero(multiplicative_order_mod(Nat.6, Nat.5))
        false
    }
    if Nat.1 < multiplicative_order_mod(Nat.6, Nat.5) {
        lte_imp_not_lt(multiplicative_order_mod(Nat.6, Nat.5), Nat.1)
        not Nat.1 < multiplicative_order_mod(Nat.6, Nat.5)
        false
    }
    multiplicative_order_mod(Nat.6, Nat.5) = Nat.1
}

/// `6` is not a primitive root modulo `5`: its order `1` is not the full
/// order `φ(5) = 4`.
theorem six_not_primitive_root_mod_five {
    not is_primitive_root_mod(Nat.6, Nat.5)
} by {
    if is_primitive_root_mod(Nat.6, Nat.5) {
        is_primitive_root_mod(Nat.6, Nat.5) =
            (Nat.5.is_prime and Nat.6.coprime(Nat.5) and
                multiplicative_order_mod(Nat.6, Nat.5) = Nat.5 - Nat.1)
        multiplicative_order_mod(Nat.6, Nat.5) = Nat.5 - Nat.1
        four_plus_one
        Nat.4 + Nat.1 = Nat.5
        add_imp_sub(Nat.4, Nat.1, Nat.5)
        Nat.5 - Nat.1 = Nat.4
        multiplicative_order_mod(Nat.6, Nat.5) = Nat.4
        order_six_mod_five
        multiplicative_order_mod(Nat.6, Nat.5) = Nat.1
        Nat.1 = Nat.4
        one_ne_four
        Nat.1 != Nat.4
        false
    }
}

/// The product `2 · 3 = 6` of the two primitive roots modulo `5` is
/// congruent to `1` modulo `5`.
theorem two_mul_three_congr_one_mod_five {
    (Nat.2 * Nat.3).congr_mod(Nat.1, Nat.5)
} by {
    nat_mul_2_3
    Nat.2 * Nat.3 = Nat.6
    congr_six_mod_five
    Nat.6.congr_mod(Nat.1, Nat.5)
    (Nat.2 * Nat.3).congr_mod(Nat.1, Nat.5)
}

/// The product of two primitive roots is not necessarily a primitive root:
/// modulo `5` both `2` and `3` are primitive roots, but their product
/// `2 · 3 = 6` is not.
theorem product_of_two_primitive_roots_not_primitive_root {
    exists(g: Nat, h: Nat) {
        is_primitive_root_mod(g, Nat.5) and is_primitive_root_mod(h, Nat.5) and
            not is_primitive_root_mod(g * h, Nat.5)
    }
} by {
    two_is_primitive_root_mod_five
    is_primitive_root_mod(Nat.2, Nat.5)
    three_is_primitive_root_mod_five
    is_primitive_root_mod(Nat.3, Nat.5)
    nat_mul_2_3
    Nat.2 * Nat.3 = Nat.6
    six_not_primitive_root_mod_five
    not is_primitive_root_mod(Nat.6, Nat.5)
    not is_primitive_root_mod(Nat.2 * Nat.3, Nat.5)
    is_primitive_root_mod(Nat.2, Nat.5) and is_primitive_root_mod(Nat.3, Nat.5) and
        not is_primitive_root_mod(Nat.2 * Nat.3, Nat.5)
    exists(g: Nat, h: Nat) {
        is_primitive_root_mod(g, Nat.5) and is_primitive_root_mod(h, Nat.5) and
            not is_primitive_root_mod(g * h, Nat.5)
    }
}

// ---------------------------------------------------------------------------
// The number of primitive roots modulo p (statement), and the case p = 7.
//
// The classical count — there are exactly φ(p - 1) primitive roots modulo
// the prime p:
//
//   theorem primitive_root_count_mod_prime(p: Nat) {
//       p.is_prime implies
//           p.range.filter(function(x: Nat) {
//               multiplicative_order_mod(x, p) = p - Nat.1
//           }).length = (p - Nat.1).totient
//   }
//
// — is left as a statement: the proof needs the identity that the exponents
// `k < p - 1` with `gcd(p - 1, k) = (p - 1)/d` number `φ(d)` for every
// divisor `d` of `p - 1`, which the library does not yet have (see the note
// in primitive_root_applications2.ac).  The case `p = 7` is verified below:
// `φ(6) = 2`, and the primitive roots modulo `7` are exactly `3` and `5`.
// ---------------------------------------------------------------------------

// --- The order of 5 modulo 7, computed from its powers. --------------------

/// `25 ≡ 4 (mod 7)`, since `25 = 3 · 7 + 4`.
theorem congr_twenty_five_mod_seven {
    Nat.25.congr_mod(Nat.4, Nat.7)
} by {
    nat_mul_3_7
    Nat.3 * Nat.7 = Nat.21
    nat_add_21_4
    Nat.21 + Nat.4 = Nat.25
    lt_four_seven
    Nat.4 < Nat.7
    mod_of_decomp(Nat.3, Nat.4, Nat.7)
    (Nat.3 * Nat.7 + Nat.4).mod(Nat.7) = Nat.4
    Nat.25.mod(Nat.7) = Nat.4
    mod_four_mod_seven
    Nat.4.mod(Nat.7) = Nat.4
    Nat.25.mod(Nat.7) = Nat.4.mod(Nat.7)
    Nat.25.congr_mod(Nat.4, Nat.7)
}

/// `20 ≡ 6 (mod 7)`, since `20 = 2 · 7 + 6`.
theorem congr_twenty_mod_seven {
    Nat.20.congr_mod(Nat.6, Nat.7)
} by {
    nat_mul_2_7
    Nat.2 * Nat.7 = Nat.14
    nat_add_14_6
    Nat.14 + Nat.6 = Nat.20
    lt_six_seven
    Nat.6 < Nat.7
    mod_of_decomp(Nat.2, Nat.6, Nat.7)
    (Nat.2 * Nat.7 + Nat.6).mod(Nat.7) = Nat.6
    Nat.20.mod(Nat.7) = Nat.6
    mod_six_mod_seven
    Nat.6.mod(Nat.7) = Nat.6
    Nat.20.mod(Nat.7) = Nat.6.mod(Nat.7)
    Nat.20.congr_mod(Nat.6, Nat.7)
}

/// `10 ≡ 3 (mod 7)`, since `10 = 1 · 7 + 3`.
theorem congr_ten_mod_seven {
    Nat.10.congr_mod(Nat.3, Nat.7)
} by {
    nat_mul_1_7
    Nat.1 * Nat.7 = Nat.7
    nat_add_7_3
    Nat.7 + Nat.3 = Nat.10
    lt_three_seven
    Nat.3 < Nat.7
    mod_of_decomp(Nat.1, Nat.3, Nat.7)
    (Nat.1 * Nat.7 + Nat.3).mod(Nat.7) = Nat.3
    Nat.10.mod(Nat.7) = Nat.3
    mod_three_mod_seven
    Nat.3.mod(Nat.7) = Nat.3
    Nat.10.mod(Nat.7) = Nat.3.mod(Nat.7)
    Nat.10.congr_mod(Nat.3, Nat.7)
}

/// `5^1 = 5`.
theorem pow_five_one {
    Nat.5.pow(Nat.1) = Nat.5
} by {
    exp_one(Nat.5)
}

/// `5^2 = 25`.
theorem pow_five_two {
    Nat.5.pow(Nat.2) = Nat.25
} by {
    exp_add(Nat.5, Nat.1, Nat.1)
    Nat.5.pow(Nat.1 + Nat.1) = Nat.5.pow(Nat.1) * Nat.5.pow(Nat.1)
    one_plus_one
    Nat.1 + Nat.1 = Nat.2
    Nat.5.pow(Nat.2) = Nat.5.pow(Nat.1) * Nat.5.pow(Nat.1)
    pow_five_one
    Nat.5.pow(Nat.1) = Nat.5
    Nat.5.pow(Nat.2) = Nat.5 * Nat.5
    nat_mul_5_5
    Nat.5 * Nat.5 = Nat.25
    Nat.5.pow(Nat.2) = Nat.25
}

/// `5^1 ≡ 5 (mod 7)`.
theorem congr_five_pow_one_mod_seven {
    Nat.5.pow(Nat.1).congr_mod(Nat.5, Nat.7)
} by {
    pow_five_one
    Nat.5.pow(Nat.1) = Nat.5
    congr_mod_refl(Nat.5, Nat.7)
    Nat.5.congr_mod(Nat.5, Nat.7)
    Nat.5.pow(Nat.1).congr_mod(Nat.5, Nat.7)
}

/// `5^2 ≡ 4 (mod 7)`.
theorem congr_five_pow_two_mod_seven {
    Nat.5.pow(Nat.2).congr_mod(Nat.4, Nat.7)
} by {
    pow_five_two
    Nat.5.pow(Nat.2) = Nat.25
    congr_twenty_five_mod_seven
    Nat.25.congr_mod(Nat.4, Nat.7)
    Nat.5.pow(Nat.2).congr_mod(Nat.4, Nat.7)
}

/// `5^3 ≡ 6 (mod 7)`.
theorem congr_five_pow_three_mod_seven {
    Nat.5.pow(Nat.3).congr_mod(Nat.6, Nat.7)
} by {
    exp_add(Nat.5, Nat.2, Nat.1)
    Nat.5.pow(Nat.2 + Nat.1) = Nat.5.pow(Nat.2) * Nat.5.pow(Nat.1)
    two_plus_one
    Nat.2 + Nat.1 = Nat.3
    Nat.5.pow(Nat.3) = Nat.5.pow(Nat.2) * Nat.5.pow(Nat.1)
    congr_five_pow_two_mod_seven
    Nat.5.pow(Nat.2).congr_mod(Nat.4, Nat.7)
    congr_five_pow_one_mod_seven
    Nat.5.pow(Nat.1).congr_mod(Nat.5, Nat.7)
    congr_mod_mul(Nat.5.pow(Nat.2), Nat.5.pow(Nat.1), Nat.4, Nat.5, Nat.7)
    (Nat.5.pow(Nat.2) * Nat.5.pow(Nat.1)).congr_mod(Nat.4 * Nat.5, Nat.7)
    Nat.5.pow(Nat.3).congr_mod(Nat.4 * Nat.5, Nat.7)
    nat_mul_4_5
    Nat.4 * Nat.5 = Nat.20
    Nat.5.pow(Nat.3).congr_mod(Nat.20, Nat.7)
    congr_twenty_mod_seven
    Nat.20.congr_mod(Nat.6, Nat.7)
    congr_mod_trans(Nat.5.pow(Nat.3), Nat.20, Nat.6, Nat.7)
    Nat.5.pow(Nat.3).congr_mod(Nat.6, Nat.7)
}

/// `6^2 = 36`.
theorem pow_six_two {
    Nat.6.pow(Nat.2) = Nat.36
} by {
    exp_add(Nat.6, Nat.1, Nat.1)
    Nat.6.pow(Nat.1 + Nat.1) = Nat.6.pow(Nat.1) * Nat.6.pow(Nat.1)
    one_plus_one
    Nat.1 + Nat.1 = Nat.2
    Nat.6.pow(Nat.2) = Nat.6.pow(Nat.1) * Nat.6.pow(Nat.1)
    exp_one(Nat.6)
    Nat.6.pow(Nat.1) = Nat.6
    Nat.6.pow(Nat.2) = Nat.6 * Nat.6
    nat_mul_6_6
    Nat.6 * Nat.6 = Nat.36
    Nat.6.pow(Nat.2) = Nat.36
}

/// `5^6 ≡ 1 (mod 7)`, since `5^6 = (5^3)^2` and `5^3 ≡ 6`, `6^2 = 36 ≡ 1`.
theorem congr_five_pow_six_mod_seven {
    Nat.5.pow(Nat.6).congr_mod(Nat.1, Nat.7)
} by {
    exp_mul(Nat.5, Nat.3, Nat.2)
    Nat.5.pow(Nat.3 * Nat.2) = Nat.5.pow(Nat.3).pow(Nat.2)
    nat_mul_3_2
    Nat.3 * Nat.2 = Nat.6
    Nat.5.pow(Nat.6) = Nat.5.pow(Nat.3).pow(Nat.2)
    congr_five_pow_three_mod_seven
    Nat.5.pow(Nat.3).congr_mod(Nat.6, Nat.7)
    congr_mod_pow(Nat.5.pow(Nat.3), Nat.6, Nat.7, Nat.2)
    Nat.5.pow(Nat.3).pow(Nat.2).congr_mod(Nat.6.pow(Nat.2), Nat.7)
    Nat.5.pow(Nat.6).congr_mod(Nat.6.pow(Nat.2), Nat.7)
    pow_six_two
    Nat.6.pow(Nat.2) = Nat.36
    Nat.5.pow(Nat.6).congr_mod(Nat.36, Nat.7)
    congr_thirty_six_mod_seven
    Nat.36.congr_mod(Nat.1, Nat.7)
    congr_mod_trans(Nat.5.pow(Nat.6), Nat.36, Nat.1, Nat.7)
    Nat.5.pow(Nat.6).congr_mod(Nat.1, Nat.7)
}

/// `5^1 ≢ 1 (mod 7)`.
theorem not_congr_five_pow_one_mod_seven {
    not Nat.5.pow(Nat.1).congr_mod(Nat.1, Nat.7)
} by {
    pow_five_one
    Nat.5.pow(Nat.1) = Nat.5
    not_congr_five_mod_seven
    if Nat.5.pow(Nat.1).congr_mod(Nat.1, Nat.7) {
        Nat.5.congr_mod(Nat.1, Nat.7)
        false
    }
}

/// `5^2 ≢ 1 (mod 7)`.
theorem not_congr_five_pow_two_mod_seven {
    not Nat.5.pow(Nat.2).congr_mod(Nat.1, Nat.7)
} by {
    congr_five_pow_two_mod_seven
    Nat.5.pow(Nat.2).congr_mod(Nat.4, Nat.7)
    not_congr_four_mod_seven
    if Nat.5.pow(Nat.2).congr_mod(Nat.1, Nat.7) {
        congr_mod_symm(Nat.5.pow(Nat.2), Nat.4, Nat.7)
        Nat.4.congr_mod(Nat.5.pow(Nat.2), Nat.7)
        congr_mod_trans(Nat.4, Nat.5.pow(Nat.2), Nat.1, Nat.7)
        Nat.4.congr_mod(Nat.1, Nat.7)
        false
    }
}

/// `5^3 ≢ 1 (mod 7)`.
theorem not_congr_five_pow_three_mod_seven {
    not Nat.5.pow(Nat.3).congr_mod(Nat.1, Nat.7)
} by {
    congr_five_pow_three_mod_seven
    Nat.5.pow(Nat.3).congr_mod(Nat.6, Nat.7)
    not_congr_six_mod_seven
    if Nat.5.pow(Nat.3).congr_mod(Nat.1, Nat.7) {
        congr_mod_symm(Nat.5.pow(Nat.3), Nat.6, Nat.7)
        Nat.6.congr_mod(Nat.5.pow(Nat.3), Nat.7)
        congr_mod_trans(Nat.6, Nat.5.pow(Nat.3), Nat.1, Nat.7)
        Nat.6.congr_mod(Nat.1, Nat.7)
        false
    }
}

/// `4^2 = 16`.
theorem pow_four_two {
    Nat.4.pow(Nat.2) = Nat.16
} by {
    exp_add(Nat.4, Nat.1, Nat.1)
    Nat.4.pow(Nat.1 + Nat.1) = Nat.4.pow(Nat.1) * Nat.4.pow(Nat.1)
    one_plus_one
    Nat.1 + Nat.1 = Nat.2
    Nat.4.pow(Nat.2) = Nat.4.pow(Nat.1) * Nat.4.pow(Nat.1)
    exp_one(Nat.4)
    Nat.4.pow(Nat.1) = Nat.4
    Nat.4.pow(Nat.2) = Nat.4 * Nat.4
    nat_mul_4_4
    Nat.4 * Nat.4 = Nat.16
    Nat.4.pow(Nat.2) = Nat.16
}

/// `16 ≡ 2 (mod 7)`, since `16 = 2 · 7 + 2`.
theorem congr_sixteen_mod_seven {
    Nat.16.congr_mod(Nat.2, Nat.7)
} by {
    nat_mul_2_7
    Nat.2 * Nat.7 = Nat.14
    nat_add_14_2
    Nat.14 + Nat.2 = Nat.16
    lt_two_seven
    Nat.2 < Nat.7
    mod_of_decomp(Nat.2, Nat.2, Nat.7)
    (Nat.2 * Nat.7 + Nat.2).mod(Nat.7) = Nat.2
    Nat.16.mod(Nat.7) = Nat.2
    mod_two_mod_seven
    Nat.2.mod(Nat.7) = Nat.2
    Nat.16.mod(Nat.7) = Nat.2.mod(Nat.7)
    Nat.16.congr_mod(Nat.2, Nat.7)
}

/// `5^4 ≡ 2 (mod 7)`.
theorem congr_five_pow_four_mod_seven {
    Nat.5.pow(Nat.4).congr_mod(Nat.2, Nat.7)
} by {
    exp_add(Nat.5, Nat.2, Nat.2)
    Nat.5.pow(Nat.2 + Nat.2) = Nat.5.pow(Nat.2) * Nat.5.pow(Nat.2)
    two_plus_two
    Nat.2 + Nat.2 = Nat.4
    Nat.5.pow(Nat.4) = Nat.5.pow(Nat.2) * Nat.5.pow(Nat.2)
    congr_five_pow_two_mod_seven
    Nat.5.pow(Nat.2).congr_mod(Nat.4, Nat.7)
    congr_mod_mul(Nat.5.pow(Nat.2), Nat.5.pow(Nat.2), Nat.4, Nat.4, Nat.7)
    (Nat.5.pow(Nat.2) * Nat.5.pow(Nat.2)).congr_mod(Nat.4 * Nat.4, Nat.7)
    Nat.5.pow(Nat.4).congr_mod(Nat.4 * Nat.4, Nat.7)
    nat_mul_4_4
    Nat.4 * Nat.4 = Nat.16
    Nat.5.pow(Nat.4).congr_mod(Nat.16, Nat.7)
    congr_sixteen_mod_seven
    Nat.16.congr_mod(Nat.2, Nat.7)
    congr_mod_trans(Nat.5.pow(Nat.4), Nat.16, Nat.2, Nat.7)
    Nat.5.pow(Nat.4).congr_mod(Nat.2, Nat.7)
}

/// `5^5 ≡ 3 (mod 7)`.
theorem congr_five_pow_five_mod_seven {
    Nat.5.pow(Nat.5).congr_mod(Nat.3, Nat.7)
} by {
    exp_add(Nat.5, Nat.4, Nat.1)
    Nat.5.pow(Nat.4 + Nat.1) = Nat.5.pow(Nat.4) * Nat.5.pow(Nat.1)
    four_plus_one
    Nat.4 + Nat.1 = Nat.5
    Nat.5.pow(Nat.5) = Nat.5.pow(Nat.4) * Nat.5.pow(Nat.1)
    congr_five_pow_four_mod_seven
    Nat.5.pow(Nat.4).congr_mod(Nat.2, Nat.7)
    congr_five_pow_one_mod_seven
    Nat.5.pow(Nat.1).congr_mod(Nat.5, Nat.7)
    congr_mod_mul(Nat.5.pow(Nat.4), Nat.5.pow(Nat.1), Nat.2, Nat.5, Nat.7)
    (Nat.5.pow(Nat.4) * Nat.5.pow(Nat.1)).congr_mod(Nat.2 * Nat.5, Nat.7)
    Nat.5.pow(Nat.5).congr_mod(Nat.2 * Nat.5, Nat.7)
    nat_mul_2_5
    Nat.2 * Nat.5 = Nat.10
    Nat.5.pow(Nat.5).congr_mod(Nat.10, Nat.7)
    congr_ten_mod_seven
    Nat.10.congr_mod(Nat.3, Nat.7)
    congr_mod_trans(Nat.5.pow(Nat.5), Nat.10, Nat.3, Nat.7)
    Nat.5.pow(Nat.5).congr_mod(Nat.3, Nat.7)
}

/// `5^4 ≢ 1 (mod 7)`.
theorem not_congr_five_pow_four_mod_seven {
    not Nat.5.pow(Nat.4).congr_mod(Nat.1, Nat.7)
} by {
    congr_five_pow_four_mod_seven
    Nat.5.pow(Nat.4).congr_mod(Nat.2, Nat.7)
    not_congr_two_mod_seven
    if Nat.5.pow(Nat.4).congr_mod(Nat.1, Nat.7) {
        congr_mod_symm(Nat.5.pow(Nat.4), Nat.2, Nat.7)
        Nat.2.congr_mod(Nat.5.pow(Nat.4), Nat.7)
        congr_mod_trans(Nat.2, Nat.5.pow(Nat.4), Nat.1, Nat.7)
        Nat.2.congr_mod(Nat.1, Nat.7)
        false
    }
}

/// `5^5 ≢ 1 (mod 7)`.
theorem not_congr_five_pow_five_mod_seven {
    not Nat.5.pow(Nat.5).congr_mod(Nat.1, Nat.7)
} by {
    congr_five_pow_five_mod_seven
    Nat.5.pow(Nat.5).congr_mod(Nat.3, Nat.7)
    not_congr_three_mod_seven
    if Nat.5.pow(Nat.5).congr_mod(Nat.1, Nat.7) {
        congr_mod_symm(Nat.5.pow(Nat.5), Nat.3, Nat.7)
        Nat.3.congr_mod(Nat.5.pow(Nat.5), Nat.7)
        congr_mod_trans(Nat.3, Nat.5.pow(Nat.5), Nat.1, Nat.7)
        Nat.3.congr_mod(Nat.1, Nat.7)
        false
    }
}

// --- The order of 5 modulo 7. ----------------------------------------------

/// `5` is coprime to `7`.
theorem five_coprime_mod_seven {
    Nat.5.coprime(Nat.7)
} by {
    seven_is_prime
    Nat.7.is_prime
    Nat.1 <= Nat.5
    lt_five_seven
    Nat.5 < Nat.7
    coprime_below_prime(Nat.7, Nat.5)
    Nat.5.coprime(Nat.7)
}

/// `5` has multiplicative order `6` modulo `7`: `5^6 ≡ 1` and no smaller
/// positive exponent gives congruence to `1`.
theorem order_five_mod_seven {
    multiplicative_order_mod(Nat.5, Nat.7) = Nat.6
} by {
    Nat.7 != Nat.0
    five_coprime_mod_seven
    Nat.5.coprime(Nat.7)
    lt_zero_six
    Nat.0 < Nat.6
    congr_five_pow_six_mod_seven
    Nat.5.pow(Nat.6).congr_mod(Nat.1, Nat.7)
    multiplicative_order_mod_minimal(Nat.5, Nat.7, Nat.6)
    multiplicative_order_mod(Nat.5, Nat.7) <= Nat.6
    multiplicative_order_mod_positive(Nat.5, Nat.7)
    Nat.0 < multiplicative_order_mod(Nat.5, Nat.7)
    if multiplicative_order_mod(Nat.5, Nat.7) = Nat.1 {
        multiplicative_order_mod_is_order(Nat.5, Nat.7)
        is_multiplicative_order_mod(Nat.5, Nat.7, multiplicative_order_mod(Nat.5, Nat.7))
        is_multiplicative_order_mod(Nat.5, Nat.7, Nat.1)
        multiplicative_order_pow_congr_one(Nat.5, Nat.7, Nat.1)
        Nat.5.pow(Nat.1).congr_mod(Nat.1, Nat.7)
        not_congr_five_pow_one_mod_seven
        false
    }
    if multiplicative_order_mod(Nat.5, Nat.7) = Nat.2 {
        multiplicative_order_mod_is_order(Nat.5, Nat.7)
        is_multiplicative_order_mod(Nat.5, Nat.7, multiplicative_order_mod(Nat.5, Nat.7))
        is_multiplicative_order_mod(Nat.5, Nat.7, Nat.2)
        multiplicative_order_pow_congr_one(Nat.5, Nat.7, Nat.2)
        Nat.5.pow(Nat.2).congr_mod(Nat.1, Nat.7)
        not_congr_five_pow_two_mod_seven
        false
    }
    if multiplicative_order_mod(Nat.5, Nat.7) = Nat.3 {
        multiplicative_order_mod_is_order(Nat.5, Nat.7)
        is_multiplicative_order_mod(Nat.5, Nat.7, multiplicative_order_mod(Nat.5, Nat.7))
        is_multiplicative_order_mod(Nat.5, Nat.7, Nat.3)
        multiplicative_order_pow_congr_one(Nat.5, Nat.7, Nat.3)
        Nat.5.pow(Nat.3).congr_mod(Nat.1, Nat.7)
        not_congr_five_pow_three_mod_seven
        false
    }
    if multiplicative_order_mod(Nat.5, Nat.7) = Nat.4 {
        multiplicative_order_mod_is_order(Nat.5, Nat.7)
        is_multiplicative_order_mod(Nat.5, Nat.7, multiplicative_order_mod(Nat.5, Nat.7))
        is_multiplicative_order_mod(Nat.5, Nat.7, Nat.4)
        multiplicative_order_pow_congr_one(Nat.5, Nat.7, Nat.4)
        Nat.5.pow(Nat.4).congr_mod(Nat.1, Nat.7)
        not_congr_five_pow_four_mod_seven
        false
    }
    if multiplicative_order_mod(Nat.5, Nat.7) = Nat.5 {
        multiplicative_order_mod_is_order(Nat.5, Nat.7)
        is_multiplicative_order_mod(Nat.5, Nat.7, multiplicative_order_mod(Nat.5, Nat.7))
        is_multiplicative_order_mod(Nat.5, Nat.7, Nat.5)
        multiplicative_order_pow_congr_one(Nat.5, Nat.7, Nat.5)
        Nat.5.pow(Nat.5).congr_mod(Nat.1, Nat.7)
        not_congr_five_pow_five_mod_seven
        false
    }
    trichotomy(multiplicative_order_mod(Nat.5, Nat.7), Nat.6)
    if multiplicative_order_mod(Nat.5, Nat.7) < Nat.6 {
        lt_suc_right(multiplicative_order_mod(Nat.5, Nat.7), Nat.5)
        if multiplicative_order_mod(Nat.5, Nat.7) = Nat.5 {
            false
        }
        multiplicative_order_mod(Nat.5, Nat.7) < Nat.5
        lt_suc_right(multiplicative_order_mod(Nat.5, Nat.7), Nat.4)
        if multiplicative_order_mod(Nat.5, Nat.7) = Nat.4 {
            false
        }
        multiplicative_order_mod(Nat.5, Nat.7) < Nat.4
        lt_suc_right(multiplicative_order_mod(Nat.5, Nat.7), Nat.3)
        if multiplicative_order_mod(Nat.5, Nat.7) = Nat.3 {
            false
        }
        multiplicative_order_mod(Nat.5, Nat.7) < Nat.3
        lt_suc_right(multiplicative_order_mod(Nat.5, Nat.7), Nat.2)
        if multiplicative_order_mod(Nat.5, Nat.7) = Nat.2 {
            false
        }
        multiplicative_order_mod(Nat.5, Nat.7) < Nat.2
        lt_suc_right(multiplicative_order_mod(Nat.5, Nat.7), Nat.1)
        if multiplicative_order_mod(Nat.5, Nat.7) = Nat.1 {
            false
        }
        multiplicative_order_mod(Nat.5, Nat.7) < Nat.1
        lt_suc_right(multiplicative_order_mod(Nat.5, Nat.7), Nat.0)
        if multiplicative_order_mod(Nat.5, Nat.7) = Nat.0 {
            Nat.0 < multiplicative_order_mod(Nat.5, Nat.7)
            Nat.0 < Nat.0
            lt_not_ref(Nat.0)
            false
        }
        multiplicative_order_mod(Nat.5, Nat.7) < Nat.0
        not_lt_zero(multiplicative_order_mod(Nat.5, Nat.7))
        false
    }
    if Nat.6 < multiplicative_order_mod(Nat.5, Nat.7) {
        lte_imp_not_lt(multiplicative_order_mod(Nat.5, Nat.7), Nat.6)
        not Nat.6 < multiplicative_order_mod(Nat.5, Nat.7)
        false
    }
    multiplicative_order_mod(Nat.5, Nat.7) = Nat.6
}

/// `5` is a primitive root modulo `7`: it is coprime to `7` and its order
/// `6` is the full order `φ(7) = 7 - 1`.
theorem five_is_primitive_root_mod_seven {
    is_primitive_root_mod(Nat.5, Nat.7)
} by {
    seven_is_prime
    Nat.7.is_prime
    five_coprime_mod_seven
    Nat.5.coprime(Nat.7)
    order_five_mod_seven
    multiplicative_order_mod(Nat.5, Nat.7) = Nat.6
    add_one_right(Nat.6)
    Nat.6 + Nat.1 = Nat.7
    add_imp_sub(Nat.6, Nat.1, Nat.7)
    Nat.7 - Nat.1 = Nat.6
    multiplicative_order_mod(Nat.5, Nat.7) = Nat.7 - Nat.1
    Nat.7.is_prime and Nat.5.coprime(Nat.7) and
        multiplicative_order_mod(Nat.5, Nat.7) = Nat.7 - Nat.1
    is_primitive_root_mod(Nat.5, Nat.7) =
        (Nat.7.is_prime and Nat.5.coprime(Nat.7) and
            multiplicative_order_mod(Nat.5, Nat.7) = Nat.7 - Nat.1)
    is_primitive_root_mod(Nat.5, Nat.7)
}

// --- The non-primitive roots modulo 7. -------------------------------------

/// `7 != 1`.
theorem seven_ne_one {
    Nat.7 != Nat.1
} by {
    lt_one_seven
    Nat.1 < Nat.7
    if Nat.7 = Nat.1 {
        Nat.1 < Nat.1
        lt_not_ref(Nat.1)
        false
    }
}

/// `0` is not coprime to `7`.
theorem zero_not_coprime_mod_seven {
    not Nat.0.coprime(Nat.7)
} by {
    if Nat.0.coprime(Nat.7) {
        coprime_zero_left_imp_one(Nat.7)
        Nat.7 = Nat.1
        seven_ne_one
        Nat.7 != Nat.1
        false
    }
}

/// `0` is not a primitive root modulo `7`.
theorem zero_not_primitive_root_mod_seven {
    not is_primitive_root_mod(Nat.0, Nat.7)
} by {
    if is_primitive_root_mod(Nat.0, Nat.7) {
        is_primitive_root_mod(Nat.0, Nat.7) =
            (Nat.7.is_prime and Nat.0.coprime(Nat.7) and
                multiplicative_order_mod(Nat.0, Nat.7) = Nat.7 - Nat.1)
        Nat.0.coprime(Nat.7)
        zero_not_coprime_mod_seven
        not Nat.0.coprime(Nat.7)
        false
    }
}

/// `1` has multiplicative order `1` modulo `7`.
theorem order_one_mod_seven {
    multiplicative_order_mod(Nat.1, Nat.7) = Nat.1
} by {
    Nat.7 != Nat.0
    coprime_one_left(Nat.7)
    Nat.1.coprime(Nat.7)
    lt_suc(Nat.0)
    Nat.0 < Nat.1
    exp_one(Nat.1)
    Nat.1.pow(Nat.1) = Nat.1
    congr_mod_refl(Nat.1, Nat.7)
    Nat.1.congr_mod(Nat.1, Nat.7)
    Nat.1.pow(Nat.1).congr_mod(Nat.1, Nat.7)
    multiplicative_order_mod_minimal(Nat.1, Nat.7, Nat.1)
    multiplicative_order_mod(Nat.1, Nat.7) <= Nat.1
    multiplicative_order_mod_positive(Nat.1, Nat.7)
    Nat.0 < multiplicative_order_mod(Nat.1, Nat.7)
    trichotomy(multiplicative_order_mod(Nat.1, Nat.7), Nat.1)
    if multiplicative_order_mod(Nat.1, Nat.7) < Nat.1 {
        lt_suc_right(multiplicative_order_mod(Nat.1, Nat.7), Nat.0)
        if multiplicative_order_mod(Nat.1, Nat.7) = Nat.0 {
            Nat.0 < multiplicative_order_mod(Nat.1, Nat.7)
            Nat.0 < Nat.0
            lt_not_ref(Nat.0)
            false
        }
        multiplicative_order_mod(Nat.1, Nat.7) < Nat.0
        not_lt_zero(multiplicative_order_mod(Nat.1, Nat.7))
        false
    }
    if Nat.1 < multiplicative_order_mod(Nat.1, Nat.7) {
        lte_imp_not_lt(multiplicative_order_mod(Nat.1, Nat.7), Nat.1)
        not Nat.1 < multiplicative_order_mod(Nat.1, Nat.7)
        false
    }
    multiplicative_order_mod(Nat.1, Nat.7) = Nat.1
}

/// `1 < 6`.
theorem lt_one_six {
    Nat.1 < Nat.6
} by {
    lt_suc(Nat.1)
    Nat.1 < Nat.2
    lt_imp_lt_suc(Nat.1, Nat.2)
    Nat.1 < Nat.3
    lt_imp_lt_suc(Nat.1, Nat.3)
    Nat.1 < Nat.4
    lt_imp_lt_suc(Nat.1, Nat.4)
    Nat.1 < Nat.5
    lt_imp_lt_suc(Nat.1, Nat.5)
    Nat.1 < Nat.6
}

/// `1 != 6`.
theorem one_ne_six {
    Nat.1 != Nat.6
} by {
    lt_one_six
    Nat.1 < Nat.6
    if Nat.1 = Nat.6 {
        Nat.1 < Nat.1
        lt_not_ref(Nat.1)
        false
    }
}

/// `1` is not a primitive root modulo `7`: its order `1` is not the full
/// order `φ(7) = 6`.
theorem one_not_primitive_root_mod_seven {
    not is_primitive_root_mod(Nat.1, Nat.7)
} by {
    if is_primitive_root_mod(Nat.1, Nat.7) {
        is_primitive_root_mod(Nat.1, Nat.7) =
            (Nat.7.is_prime and Nat.1.coprime(Nat.7) and
                multiplicative_order_mod(Nat.1, Nat.7) = Nat.7 - Nat.1)
        multiplicative_order_mod(Nat.1, Nat.7) = Nat.7 - Nat.1
        add_one_right(Nat.6)
        Nat.6 + Nat.1 = Nat.7
        add_imp_sub(Nat.6, Nat.1, Nat.7)
        Nat.7 - Nat.1 = Nat.6
        multiplicative_order_mod(Nat.1, Nat.7) = Nat.6
        order_one_mod_seven
        multiplicative_order_mod(Nat.1, Nat.7) = Nat.1
        Nat.1 = Nat.6
        one_ne_six
        Nat.1 != Nat.6
        false
    }
}

/// `8 ≡ 1 (mod 7)`, since `8 = 1 · 7 + 1`.
theorem congr_eight_mod_seven {
    Nat.8.congr_mod(Nat.1, Nat.7)
} by {
    nat_mul_1_7
    Nat.1 * Nat.7 = Nat.7
    nat_add_7_1
    Nat.7 + Nat.1 = Nat.8
    lt_one_seven
    Nat.1 < Nat.7
    mod_of_decomp(Nat.1, Nat.1, Nat.7)
    (Nat.1 * Nat.7 + Nat.1).mod(Nat.7) = Nat.1
    Nat.8.mod(Nat.7) = Nat.1
    mod_one_mod_seven
    Nat.1.mod(Nat.7) = Nat.1
    Nat.8.mod(Nat.7) = Nat.1.mod(Nat.7)
    Nat.8.congr_mod(Nat.1, Nat.7)
}

/// `2^3 = 8 ≡ 1 (mod 7)`.
theorem congr_two_pow_three_mod_seven {
    Nat.2.pow(Nat.3).congr_mod(Nat.1, Nat.7)
} by {
    pow_two_three
    Nat.2.pow(Nat.3) = Nat.8
    congr_eight_mod_seven
    Nat.8.congr_mod(Nat.1, Nat.7)
    Nat.2.pow(Nat.3).congr_mod(Nat.1, Nat.7)
}

/// `2^2 = 4 ≢ 1 (mod 7)`.
theorem not_congr_two_pow_two_mod_seven {
    not Nat.2.pow(Nat.2).congr_mod(Nat.1, Nat.7)
} by {
    pow_two_two
    Nat.2.pow(Nat.2) = Nat.4
    not_congr_four_mod_seven
    if Nat.2.pow(Nat.2).congr_mod(Nat.1, Nat.7) {
        Nat.4.congr_mod(Nat.1, Nat.7)
        false
    }
}

/// `2` is coprime to `7`.
theorem two_coprime_mod_seven {
    Nat.2.coprime(Nat.7)
} by {
    seven_is_prime
    Nat.7.is_prime
    Nat.1 <= Nat.2
    lt_two_seven
    Nat.2 < Nat.7
    coprime_below_prime(Nat.7, Nat.2)
    Nat.2.coprime(Nat.7)
}

/// `2` has multiplicative order `3` modulo `7`: `2^3 ≡ 1` but `2^1 ≢ 1` and
/// `2^2 ≢ 1`.
theorem order_two_mod_seven {
    multiplicative_order_mod(Nat.2, Nat.7) = Nat.3
} by {
    Nat.7 != Nat.0
    two_coprime_mod_seven
    Nat.2.coprime(Nat.7)
    lt_zero_three
    Nat.0 < Nat.3
    congr_two_pow_three_mod_seven
    Nat.2.pow(Nat.3).congr_mod(Nat.1, Nat.7)
    multiplicative_order_mod_minimal(Nat.2, Nat.7, Nat.3)
    multiplicative_order_mod(Nat.2, Nat.7) <= Nat.3
    multiplicative_order_mod_positive(Nat.2, Nat.7)
    Nat.0 < multiplicative_order_mod(Nat.2, Nat.7)
    if multiplicative_order_mod(Nat.2, Nat.7) = Nat.1 {
        multiplicative_order_mod_is_order(Nat.2, Nat.7)
        is_multiplicative_order_mod(Nat.2, Nat.7, multiplicative_order_mod(Nat.2, Nat.7))
        is_multiplicative_order_mod(Nat.2, Nat.7, Nat.1)
        multiplicative_order_pow_congr_one(Nat.2, Nat.7, Nat.1)
        Nat.2.pow(Nat.1).congr_mod(Nat.1, Nat.7)
        pow_two_one
        Nat.2.pow(Nat.1) = Nat.2
        Nat.2.congr_mod(Nat.1, Nat.7)
        not_congr_two_mod_seven
        false
    }
    if multiplicative_order_mod(Nat.2, Nat.7) = Nat.2 {
        multiplicative_order_mod_is_order(Nat.2, Nat.7)
        is_multiplicative_order_mod(Nat.2, Nat.7, multiplicative_order_mod(Nat.2, Nat.7))
        is_multiplicative_order_mod(Nat.2, Nat.7, Nat.2)
        multiplicative_order_pow_congr_one(Nat.2, Nat.7, Nat.2)
        Nat.2.pow(Nat.2).congr_mod(Nat.1, Nat.7)
        not_congr_two_pow_two_mod_seven
        false
    }
    trichotomy(multiplicative_order_mod(Nat.2, Nat.7), Nat.3)
    if multiplicative_order_mod(Nat.2, Nat.7) < Nat.3 {
        lt_suc_right(multiplicative_order_mod(Nat.2, Nat.7), Nat.2)
        if multiplicative_order_mod(Nat.2, Nat.7) = Nat.2 {
            false
        }
        multiplicative_order_mod(Nat.2, Nat.7) < Nat.2
        lt_suc_right(multiplicative_order_mod(Nat.2, Nat.7), Nat.1)
        if multiplicative_order_mod(Nat.2, Nat.7) = Nat.1 {
            false
        }
        multiplicative_order_mod(Nat.2, Nat.7) < Nat.1
        lt_suc_right(multiplicative_order_mod(Nat.2, Nat.7), Nat.0)
        if multiplicative_order_mod(Nat.2, Nat.7) = Nat.0 {
            Nat.0 < multiplicative_order_mod(Nat.2, Nat.7)
            Nat.0 < Nat.0
            lt_not_ref(Nat.0)
            false
        }
        multiplicative_order_mod(Nat.2, Nat.7) < Nat.0
        not_lt_zero(multiplicative_order_mod(Nat.2, Nat.7))
        false
    }
    if Nat.3 < multiplicative_order_mod(Nat.2, Nat.7) {
        lte_imp_not_lt(multiplicative_order_mod(Nat.2, Nat.7), Nat.3)
        not Nat.3 < multiplicative_order_mod(Nat.2, Nat.7)
        false
    }
    multiplicative_order_mod(Nat.2, Nat.7) = Nat.3
}

/// `2 < 6`.
theorem lt_two_six {
    Nat.2 < Nat.6
} by {
    lt_suc(Nat.2)
    Nat.2 < Nat.3
    lt_imp_lt_suc(Nat.2, Nat.3)
    Nat.2 < Nat.4
    lt_imp_lt_suc(Nat.2, Nat.4)
    Nat.2 < Nat.5
    lt_imp_lt_suc(Nat.2, Nat.5)
    Nat.2 < Nat.6
}

/// `3 < 6`.
theorem lt_three_six {
    Nat.3 < Nat.6
} by {
    lt_suc(Nat.3)
    Nat.3 < Nat.4
    lt_imp_lt_suc(Nat.3, Nat.4)
    Nat.3 < Nat.5
    lt_imp_lt_suc(Nat.3, Nat.5)
    Nat.3 < Nat.6
}

/// `2 != 6`.
theorem two_ne_six {
    Nat.2 != Nat.6
} by {
    lt_two_six
    Nat.2 < Nat.6
    if Nat.2 = Nat.6 {
        Nat.2 < Nat.2
        lt_not_ref(Nat.2)
        false
    }
}

/// `3 != 6`.
theorem three_ne_six {
    Nat.3 != Nat.6
} by {
    lt_three_six
    Nat.3 < Nat.6
    if Nat.3 = Nat.6 {
        Nat.3 < Nat.3
        lt_not_ref(Nat.3)
        false
    }
}

/// `2` is not a primitive root modulo `7`: its order `3` is not the full
/// order `φ(7) = 6`.
theorem two_not_primitive_root_mod_seven {
    not is_primitive_root_mod(Nat.2, Nat.7)
} by {
    if is_primitive_root_mod(Nat.2, Nat.7) {
        is_primitive_root_mod(Nat.2, Nat.7) =
            (Nat.7.is_prime and Nat.2.coprime(Nat.7) and
                multiplicative_order_mod(Nat.2, Nat.7) = Nat.7 - Nat.1)
        multiplicative_order_mod(Nat.2, Nat.7) = Nat.7 - Nat.1
        add_one_right(Nat.6)
        Nat.6 + Nat.1 = Nat.7
        add_imp_sub(Nat.6, Nat.1, Nat.7)
        Nat.7 - Nat.1 = Nat.6
        multiplicative_order_mod(Nat.2, Nat.7) = Nat.6
        order_two_mod_seven
        multiplicative_order_mod(Nat.2, Nat.7) = Nat.3
        Nat.3 = Nat.6
        three_ne_six
        Nat.3 != Nat.6
        false
    }
}

/// `4^3 ≡ 1 (mod 7)`, since `4 = 2^2` and `2^6 = (2^3)^2 ≡ 1`.
theorem congr_four_pow_three_mod_seven {
    Nat.4.pow(Nat.3).congr_mod(Nat.1, Nat.7)
} by {
    pow_two_two
    Nat.2.pow(Nat.2) = Nat.4
    exp_mul(Nat.2, Nat.2, Nat.3)
    Nat.2.pow(Nat.2 * Nat.3) = Nat.2.pow(Nat.2).pow(Nat.3)
    nat_mul_2_3
    Nat.2 * Nat.3 = Nat.6
    Nat.2.pow(Nat.6) = Nat.2.pow(Nat.2).pow(Nat.3)
    Nat.4.pow(Nat.3) = Nat.2.pow(Nat.2).pow(Nat.3)
    congr_two_pow_three_mod_seven
    Nat.2.pow(Nat.3).congr_mod(Nat.1, Nat.7)
    congr_mod_pow(Nat.2.pow(Nat.3), Nat.1, Nat.7, Nat.2)
    Nat.2.pow(Nat.3).pow(Nat.2).congr_mod(Nat.1.pow(Nat.2), Nat.7)
    exp_mul(Nat.2, Nat.3, Nat.2)
    Nat.2.pow(Nat.3 * Nat.2) = Nat.2.pow(Nat.3).pow(Nat.2)
    nat_mul_3_2
    Nat.3 * Nat.2 = Nat.6
    Nat.2.pow(Nat.6) = Nat.2.pow(Nat.3).pow(Nat.2)
    one_exp(Nat.2)
    Nat.1.pow(Nat.2) = Nat.1
    Nat.2.pow(Nat.6).congr_mod(Nat.1, Nat.7)
    Nat.4.pow(Nat.3).congr_mod(Nat.2.pow(Nat.6), Nat.7)
    congr_mod_trans(Nat.4.pow(Nat.3), Nat.2.pow(Nat.6), Nat.1, Nat.7)
    Nat.4.pow(Nat.3).congr_mod(Nat.1, Nat.7)
}

/// `4^1 = 4 ≢ 1 (mod 7)`.
theorem not_congr_four_pow_one_mod_seven {
    not Nat.4.pow(Nat.1).congr_mod(Nat.1, Nat.7)
} by {
    exp_one(Nat.4)
    Nat.4.pow(Nat.1) = Nat.4
    not_congr_four_mod_seven
    if Nat.4.pow(Nat.1).congr_mod(Nat.1, Nat.7) {
        Nat.4.congr_mod(Nat.1, Nat.7)
        false
    }
}

/// `4^2 = 16 ≡ 2 ≢ 1 (mod 7)`.
theorem not_congr_four_pow_two_mod_seven {
    not Nat.4.pow(Nat.2).congr_mod(Nat.1, Nat.7)
} by {
    pow_four_two
    Nat.4.pow(Nat.2) = Nat.16
    congr_sixteen_mod_seven
    Nat.16.congr_mod(Nat.2, Nat.7)
    if Nat.4.pow(Nat.2).congr_mod(Nat.1, Nat.7) {
        congr_mod_symm(Nat.4.pow(Nat.2), Nat.2, Nat.7)
        Nat.2.congr_mod(Nat.4.pow(Nat.2), Nat.7)
        congr_mod_trans(Nat.2, Nat.4.pow(Nat.2), Nat.1, Nat.7)
        Nat.2.congr_mod(Nat.1, Nat.7)
        not_congr_two_mod_seven
        false
    }
}

/// `4` is coprime to `7`.
theorem four_coprime_mod_seven {
    Nat.4.coprime(Nat.7)
} by {
    seven_is_prime
    Nat.7.is_prime
    Nat.1 <= Nat.4
    lt_four_seven
    Nat.4 < Nat.7
    coprime_below_prime(Nat.7, Nat.4)
    Nat.4.coprime(Nat.7)
}

/// `4` has multiplicative order `3` modulo `7`: `4^3 ≡ 1` but `4^1 ≢ 1` and
/// `4^2 ≢ 1`.
theorem order_four_mod_seven {
    multiplicative_order_mod(Nat.4, Nat.7) = Nat.3
} by {
    Nat.7 != Nat.0
    four_coprime_mod_seven
    Nat.4.coprime(Nat.7)
    lt_zero_three
    Nat.0 < Nat.3
    congr_four_pow_three_mod_seven
    Nat.4.pow(Nat.3).congr_mod(Nat.1, Nat.7)
    multiplicative_order_mod_minimal(Nat.4, Nat.7, Nat.3)
    multiplicative_order_mod(Nat.4, Nat.7) <= Nat.3
    multiplicative_order_mod_positive(Nat.4, Nat.7)
    Nat.0 < multiplicative_order_mod(Nat.4, Nat.7)
    if multiplicative_order_mod(Nat.4, Nat.7) = Nat.1 {
        multiplicative_order_mod_is_order(Nat.4, Nat.7)
        is_multiplicative_order_mod(Nat.4, Nat.7, multiplicative_order_mod(Nat.4, Nat.7))
        is_multiplicative_order_mod(Nat.4, Nat.7, Nat.1)
        multiplicative_order_pow_congr_one(Nat.4, Nat.7, Nat.1)
        Nat.4.pow(Nat.1).congr_mod(Nat.1, Nat.7)
        not_congr_four_pow_one_mod_seven
        false
    }
    if multiplicative_order_mod(Nat.4, Nat.7) = Nat.2 {
        multiplicative_order_mod_is_order(Nat.4, Nat.7)
        is_multiplicative_order_mod(Nat.4, Nat.7, multiplicative_order_mod(Nat.4, Nat.7))
        is_multiplicative_order_mod(Nat.4, Nat.7, Nat.2)
        multiplicative_order_pow_congr_one(Nat.4, Nat.7, Nat.2)
        Nat.4.pow(Nat.2).congr_mod(Nat.1, Nat.7)
        not_congr_four_pow_two_mod_seven
        false
    }
    trichotomy(multiplicative_order_mod(Nat.4, Nat.7), Nat.3)
    if multiplicative_order_mod(Nat.4, Nat.7) < Nat.3 {
        lt_suc_right(multiplicative_order_mod(Nat.4, Nat.7), Nat.2)
        if multiplicative_order_mod(Nat.4, Nat.7) = Nat.2 {
            false
        }
        multiplicative_order_mod(Nat.4, Nat.7) < Nat.2
        lt_suc_right(multiplicative_order_mod(Nat.4, Nat.7), Nat.1)
        if multiplicative_order_mod(Nat.4, Nat.7) = Nat.1 {
            false
        }
        multiplicative_order_mod(Nat.4, Nat.7) < Nat.1
        lt_suc_right(multiplicative_order_mod(Nat.4, Nat.7), Nat.0)
        if multiplicative_order_mod(Nat.4, Nat.7) = Nat.0 {
            Nat.0 < multiplicative_order_mod(Nat.4, Nat.7)
            Nat.0 < Nat.0
            lt_not_ref(Nat.0)
            false
        }
        multiplicative_order_mod(Nat.4, Nat.7) < Nat.0
        not_lt_zero(multiplicative_order_mod(Nat.4, Nat.7))
        false
    }
    if Nat.3 < multiplicative_order_mod(Nat.4, Nat.7) {
        lte_imp_not_lt(multiplicative_order_mod(Nat.4, Nat.7), Nat.3)
        not Nat.3 < multiplicative_order_mod(Nat.4, Nat.7)
        false
    }
    multiplicative_order_mod(Nat.4, Nat.7) = Nat.3
}

/// `4` is not a primitive root modulo `7`: its order `3` is not the full
/// order `φ(7) = 6`.
theorem four_not_primitive_root_mod_seven {
    not is_primitive_root_mod(Nat.4, Nat.7)
} by {
    if is_primitive_root_mod(Nat.4, Nat.7) {
        is_primitive_root_mod(Nat.4, Nat.7) =
            (Nat.7.is_prime and Nat.4.coprime(Nat.7) and
                multiplicative_order_mod(Nat.4, Nat.7) = Nat.7 - Nat.1)
        multiplicative_order_mod(Nat.4, Nat.7) = Nat.7 - Nat.1
        add_one_right(Nat.6)
        Nat.6 + Nat.1 = Nat.7
        add_imp_sub(Nat.6, Nat.1, Nat.7)
        Nat.7 - Nat.1 = Nat.6
        multiplicative_order_mod(Nat.4, Nat.7) = Nat.6
        order_four_mod_seven
        multiplicative_order_mod(Nat.4, Nat.7) = Nat.3
        Nat.3 = Nat.6
        three_ne_six
        Nat.3 != Nat.6
        false
    }
}

/// `6^2 = 36 ≡ 1 (mod 7)`.
theorem congr_six_pow_two_mod_seven {
    Nat.6.pow(Nat.2).congr_mod(Nat.1, Nat.7)
} by {
    pow_six_two
    Nat.6.pow(Nat.2) = Nat.36
    congr_thirty_six_mod_seven
    Nat.36.congr_mod(Nat.1, Nat.7)
    Nat.6.pow(Nat.2).congr_mod(Nat.1, Nat.7)
}

/// `6^1 = 6 ≢ 1 (mod 7)`.
theorem not_congr_six_pow_one_mod_seven {
    not Nat.6.pow(Nat.1).congr_mod(Nat.1, Nat.7)
} by {
    exp_one(Nat.6)
    Nat.6.pow(Nat.1) = Nat.6
    not_congr_six_mod_seven
    if Nat.6.pow(Nat.1).congr_mod(Nat.1, Nat.7) {
        Nat.6.congr_mod(Nat.1, Nat.7)
        false
    }
}

/// `6` is coprime to `7`.
theorem six_coprime_mod_seven {
    Nat.6.coprime(Nat.7)
} by {
    seven_is_prime
    Nat.7.is_prime
    Nat.1 <= Nat.6
    lt_six_seven
    Nat.6 < Nat.7
    coprime_below_prime(Nat.7, Nat.6)
    Nat.6.coprime(Nat.7)
}

/// `6` has multiplicative order `2` modulo `7`: `6^2 ≡ 1` but `6^1 ≢ 1`.
theorem order_six_mod_seven {
    multiplicative_order_mod(Nat.6, Nat.7) = Nat.2
} by {
    Nat.7 != Nat.0
    six_coprime_mod_seven
    Nat.6.coprime(Nat.7)
    lt_zero_two
    Nat.0 < Nat.2
    congr_six_pow_two_mod_seven
    Nat.6.pow(Nat.2).congr_mod(Nat.1, Nat.7)
    multiplicative_order_mod_minimal(Nat.6, Nat.7, Nat.2)
    multiplicative_order_mod(Nat.6, Nat.7) <= Nat.2
    multiplicative_order_mod_positive(Nat.6, Nat.7)
    Nat.0 < multiplicative_order_mod(Nat.6, Nat.7)
    if multiplicative_order_mod(Nat.6, Nat.7) = Nat.1 {
        multiplicative_order_mod_is_order(Nat.6, Nat.7)
        is_multiplicative_order_mod(Nat.6, Nat.7, multiplicative_order_mod(Nat.6, Nat.7))
        is_multiplicative_order_mod(Nat.6, Nat.7, Nat.1)
        multiplicative_order_pow_congr_one(Nat.6, Nat.7, Nat.1)
        Nat.6.pow(Nat.1).congr_mod(Nat.1, Nat.7)
        not_congr_six_pow_one_mod_seven
        false
    }
    trichotomy(multiplicative_order_mod(Nat.6, Nat.7), Nat.2)
    if multiplicative_order_mod(Nat.6, Nat.7) < Nat.2 {
        lt_suc_right(multiplicative_order_mod(Nat.6, Nat.7), Nat.1)
        if multiplicative_order_mod(Nat.6, Nat.7) = Nat.1 {
            false
        }
        multiplicative_order_mod(Nat.6, Nat.7) < Nat.1
        lt_suc_right(multiplicative_order_mod(Nat.6, Nat.7), Nat.0)
        if multiplicative_order_mod(Nat.6, Nat.7) = Nat.0 {
            Nat.0 < multiplicative_order_mod(Nat.6, Nat.7)
            Nat.0 < Nat.0
            lt_not_ref(Nat.0)
            false
        }
        multiplicative_order_mod(Nat.6, Nat.7) < Nat.0
        not_lt_zero(multiplicative_order_mod(Nat.6, Nat.7))
        false
    }
    if Nat.2 < multiplicative_order_mod(Nat.6, Nat.7) {
        lte_imp_not_lt(multiplicative_order_mod(Nat.6, Nat.7), Nat.2)
        not Nat.2 < multiplicative_order_mod(Nat.6, Nat.7)
        false
    }
    multiplicative_order_mod(Nat.6, Nat.7) = Nat.2
}

/// `6` is not a primitive root modulo `7`: its order `2` is not the full
/// order `φ(7) = 6`.
theorem six_not_primitive_root_mod_seven {
    not is_primitive_root_mod(Nat.6, Nat.7)
} by {
    if is_primitive_root_mod(Nat.6, Nat.7) {
        is_primitive_root_mod(Nat.6, Nat.7) =
            (Nat.7.is_prime and Nat.6.coprime(Nat.7) and
                multiplicative_order_mod(Nat.6, Nat.7) = Nat.7 - Nat.1)
        multiplicative_order_mod(Nat.6, Nat.7) = Nat.7 - Nat.1
        add_one_right(Nat.6)
        Nat.6 + Nat.1 = Nat.7
        add_imp_sub(Nat.6, Nat.1, Nat.7)
        Nat.7 - Nat.1 = Nat.6
        multiplicative_order_mod(Nat.6, Nat.7) = Nat.6
        order_six_mod_seven
        multiplicative_order_mod(Nat.6, Nat.7) = Nat.2
        Nat.2 = Nat.6
        two_ne_six
        Nat.2 != Nat.6
        false
    }
}

// --- The classification of primitive roots modulo 7. ------------------------

/// `x < 7` forces `x` into `{0, 1, 2, 3, 4, 5, 6}`.
theorem k_below_seven_cases(k: Nat) {
    k < Nat.7 implies (k = Nat.0 or k = Nat.1 or k = Nat.2 or k = Nat.3 or
        k = Nat.4 or k = Nat.5 or k = Nat.6)
} by {
    if k < Nat.7 {
        lt_suc_right(k, Nat.6)
        k = Nat.6 or k < Nat.6
        if k < Nat.6 {
            lt_suc_right(k, Nat.5)
            k = Nat.5 or k < Nat.5
            if k < Nat.5 {
                lt_suc_right(k, Nat.4)
                k = Nat.4 or k < Nat.4
                if k < Nat.4 {
                    lt_suc_right(k, Nat.3)
                    k = Nat.3 or k < Nat.3
                    if k < Nat.3 {
                        lt_suc_right(k, Nat.2)
                        k = Nat.2 or k < Nat.2
                        if k < Nat.2 {
                            lt_suc_right(k, Nat.1)
                            k = Nat.1 or k < Nat.1
                            if k < Nat.1 {
                                lt_suc_right(k, Nat.0)
                                k = Nat.0 or k < Nat.0
                                if k < Nat.0 {
                                    not_lt_zero(k)
                                    false
                                } else {
                                    k = Nat.0
                                    k = Nat.0 or k = Nat.1 or k = Nat.2 or k = Nat.3 or
                                        k = Nat.4 or k = Nat.5 or k = Nat.6
                                }
                            } else {
                                k = Nat.1
                                k = Nat.0 or k = Nat.1 or k = Nat.2 or k = Nat.3 or
                                    k = Nat.4 or k = Nat.5 or k = Nat.6
                            }
                        } else {
                            k = Nat.2
                            k = Nat.0 or k = Nat.1 or k = Nat.2 or k = Nat.3 or
                                k = Nat.4 or k = Nat.5 or k = Nat.6
                        }
                    } else {
                        k = Nat.3
                        k = Nat.0 or k = Nat.1 or k = Nat.2 or k = Nat.3 or
                            k = Nat.4 or k = Nat.5 or k = Nat.6
                    }
                } else {
                    k = Nat.4
                    k = Nat.0 or k = Nat.1 or k = Nat.2 or k = Nat.3 or
                        k = Nat.4 or k = Nat.5 or k = Nat.6
                }
            } else {
                k = Nat.5
                k = Nat.0 or k = Nat.1 or k = Nat.2 or k = Nat.3 or
                    k = Nat.4 or k = Nat.5 or k = Nat.6
            }
        } else {
            k = Nat.6
            k = Nat.0 or k = Nat.1 or k = Nat.2 or k = Nat.3 or
                k = Nat.4 or k = Nat.5 or k = Nat.6
        }
    }
}

/// A primitive root below `7` is either `3` or `5`.
theorem primitive_root_mod_seven_imp_three_or_five(x: Nat) {
    x < Nat.7 implies (is_primitive_root_mod(x, Nat.7) implies (x = Nat.3 or x = Nat.5))
} by {
    if x < Nat.7 {
        if is_primitive_root_mod(x, Nat.7) {
            k_below_seven_cases(x)
            x = Nat.0 or x = Nat.1 or x = Nat.2 or x = Nat.3 or
                x = Nat.4 or x = Nat.5 or x = Nat.6
            if x = Nat.0 {
                zero_not_primitive_root_mod_seven
                not is_primitive_root_mod(Nat.0, Nat.7)
                false
            }
            if x = Nat.1 {
                one_not_primitive_root_mod_seven
                not is_primitive_root_mod(Nat.1, Nat.7)
                false
            }
            if x = Nat.2 {
                two_not_primitive_root_mod_seven
                not is_primitive_root_mod(Nat.2, Nat.7)
                false
            }
            if x = Nat.4 {
                four_not_primitive_root_mod_seven
                not is_primitive_root_mod(Nat.4, Nat.7)
                false
            }
            if x = Nat.6 {
                six_not_primitive_root_mod_seven
                not is_primitive_root_mod(Nat.6, Nat.7)
                false
            }
            x = Nat.3 or x = Nat.5
        }
    }
}

/// Every `x` below `7` equal to `3` or `5` is a primitive root modulo `7`.
theorem three_or_five_imp_primitive_root_mod_seven(x: Nat) {
    x < Nat.7 implies ((x = Nat.3 or x = Nat.5) implies is_primitive_root_mod(x, Nat.7))
} by {
    if x < Nat.7 {
        if x = Nat.3 or x = Nat.5 {
            if x = Nat.3 {
                three_is_primitive_root_mod_seven
                is_primitive_root_mod(Nat.3, Nat.7)
                is_primitive_root_mod(x, Nat.7)
            } else {
                five_is_primitive_root_mod_seven
                is_primitive_root_mod(Nat.5, Nat.7)
                is_primitive_root_mod(x, Nat.7)
            }
            is_primitive_root_mod(x, Nat.7)
        }
    }
}

/// The primitive roots modulo `7` are exactly `3` and `5`.
theorem primitive_root_mod_seven_iff_three_or_five(x: Nat) {
    x < Nat.7 implies (is_primitive_root_mod(x, Nat.7) = (x = Nat.3 or x = Nat.5))
} by {
    if x < Nat.7 {
        if is_primitive_root_mod(x, Nat.7) {
            primitive_root_mod_seven_imp_three_or_five(x)
            x = Nat.3 or x = Nat.5
            eq_true_intro(is_primitive_root_mod(x, Nat.7))
            is_primitive_root_mod(x, Nat.7) = true
            eq_true_intro(x = Nat.3 or x = Nat.5)
            (x = Nat.3 or x = Nat.5) = true
            is_primitive_root_mod(x, Nat.7) = (x = Nat.3 or x = Nat.5)
        }
        if not is_primitive_root_mod(x, Nat.7) {
            if x = Nat.3 or x = Nat.5 {
                three_or_five_imp_primitive_root_mod_seven(x)
                is_primitive_root_mod(x, Nat.7)
                false
            }
            not (x = Nat.3 or x = Nat.5)
            eq_false_intro(is_primitive_root_mod(x, Nat.7))
            is_primitive_root_mod(x, Nat.7) = false
            eq_false_intro(x = Nat.3 or x = Nat.5)
            (x = Nat.3 or x = Nat.5) = false
            is_primitive_root_mod(x, Nat.7) = (x = Nat.3 or x = Nat.5)
        }
        is_primitive_root_mod(x, Nat.7) = (x = Nat.3 or x = Nat.5)
    }
}

// --- The totient of 6. ------------------------------------------------------

/// `φ(6) = 2`: Euler's totient of `6` is `2`, since `6 = 2 · 3` and
/// `φ(2 · 3) = (2 - 1)(3 - 1)`.
theorem totient_six_is_two {
    Nat.6.totient = Nat.2
} by {
    two_is_prime
    Nat.2.is_prime
    three_is_prime
    Nat.3.is_prime
    lt_two_three
    Nat.2 < Nat.3
    Nat.2 != Nat.3
    totient_pq(Nat.2, Nat.3)
    (Nat.2 * Nat.3).totient = (Nat.2 - Nat.1) * (Nat.3 - Nat.1)
    nat_mul_2_3
    Nat.2 * Nat.3 = Nat.6
    Nat.6.totient = (Nat.2 - Nat.1) * (Nat.3 - Nat.1)
    suc_sub_one(Nat.1)
    Nat.2 - Nat.1 = Nat.1
    suc_sub_one(Nat.2)
    Nat.3 - Nat.1 = Nat.2
    (Nat.2 - Nat.1) * (Nat.3 - Nat.1) = Nat.1 * Nat.2
    nat_mul_1_2
    Nat.1 * Nat.2 = Nat.2
    Nat.6.totient = Nat.2
}

/// `φ(7 - 1) = φ(6) = 2`.
theorem totient_of_seven_minus_one {
    (Nat.7 - Nat.1).totient = Nat.2
} by {
    add_one_right(Nat.6)
    Nat.6 + Nat.1 = Nat.7
    add_imp_sub(Nat.6, Nat.1, Nat.7)
    Nat.7 - Nat.1 = Nat.6
    (Nat.7 - Nat.1).totient = Nat.6.totient
    totient_six_is_two
    Nat.6.totient = Nat.2
    (Nat.7 - Nat.1).totient = Nat.2
}
