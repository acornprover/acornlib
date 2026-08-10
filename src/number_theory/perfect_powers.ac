/// Perfect powers.
///
/// A perfect power is a natural number of the form a^k with exponent k >= 2:
/// the squares, cubes, fourth powers, and so on.  This file develops the
/// basic theory built on the library's exponentiation (nat/) and prime
/// factorisation (factorisation.ac):
///
///   1. The small perfect powers 4 = 2², 8 = 2³, and 9 = 3², together with
///      the exponentiation computations 2² = 4, 2³ = 8, 3² = 9 (Section 1).
///
///   2. The exponent law a^m · a^n = a^(m + n), the identity behind products
///      of powers with a common base, and the corollary that such a product
///      is again a perfect power when m + n >= 2 (Section 2).
///
///   3. The non-perfect-powers 2 and 6 (Section 3).  The proof runs through
///      the prime-multiplicity function of factorisation.ac: in a perfect
///      power a^k every prime occurs with multiplicity exactly k times its
///      multiplicity in a (the p-adic valuation identity v_p(a^k) = k·v_p(a)),
///      so every prime multiplicity of a perfect power is a multiple of the
///      exponent k.  But the prime two occurs with multiplicity exactly one
///      in both 2 and 6 = 2·3; a multiple of k >= 2 is never one, forcing
///      k = 1, contrary to the definition.
///
///   4. Catalan's conjecture — the only consecutive perfect powers are
///      8 = 2³ and 9 = 3² — is recorded as a statement (Section 4).  The
///      proof (Mihăilescu, 2002) is deep and is not formalized here.
from nat import Nat, sq_eq_mul, nat_mul_2_2, nat_mul_2_3, nat_mul_2_4, nat_mul_3_3,
    pow_add, pow_zero, exp_ne_zero, zero_exp, lte_imp_not_lt, lt_and_lte, not_lt_zero,
    lt_suc
from number_theory.factorisation import count_prime_factor, count_prime_factor_mul,
    count_prime_factor_self, count_prime_factor_other_prime, count_prime_factor_one
from number_theory.coprime import nat_divides_one_imp_one
from number_theory.falling_product import nat_two_prime
from number_theory.goldbach import three_is_prime
numerals Nat

// ============================================================================
// Definition
// ============================================================================

/// True when n is a perfect power: n = a^k for some base a and exponent
/// k >= 2.
define is_perfect_power(n: Nat) -> Bool {
    exists(a: Nat, k: Nat) {
        Nat.2 <= k and n = a.pow(k)
    }
}

/// The defining equation of perfect-powerness.
theorem is_perfect_power_eq(n: Nat) {
    is_perfect_power(n) = exists(a: Nat, k: Nat) {
        Nat.2 <= k and n = a.pow(k)
    }
}

/// A base and an exponent at least two witness a perfect power.
theorem is_perfect_power_intro(n: Nat, a: Nat, k: Nat) {
    Nat.2 <= k and n = a.pow(k) implies is_perfect_power(n)
} by {
    if Nat.2 <= k and n = a.pow(k) {
        is_perfect_power_eq(n)
        exists(a2: Nat, k2: Nat) {
            Nat.2 <= k2 and n = a2.pow(k2)
        }
        is_perfect_power(n)
    }
}

/// A perfect power decomposes into a base and an exponent at least two.
theorem is_perfect_power_apply(n: Nat) {
    is_perfect_power(n) implies exists(a: Nat, k: Nat) {
        Nat.2 <= k and n = a.pow(k)
    }
} by {
    if is_perfect_power(n) {
        is_perfect_power_eq(n)
        exists(a: Nat, k: Nat) {
            Nat.2 <= k and n = a.pow(k)
        }
    }
}

// ============================================================================
// Section 1: the small perfect powers
// ============================================================================

/// The square of two: 2² = 4.
theorem two_squared_is_four {
    Nat.2.pow(Nat.2) = Nat.4
} by {
    sq_eq_mul(Nat.2)
    Nat.2.pow(Nat.2) = Nat.2 * Nat.2
    nat_mul_2_2
    Nat.2 * Nat.2 = Nat.4
    Nat.2.pow(Nat.2) = Nat.4
}

/// The cube of two: 2³ = 8.
theorem two_cubed_is_eight {
    Nat.2.pow(Nat.3) = Nat.8
} by {
    Nat.2.pow(Nat.3) = Nat.2 * Nat.2.pow(Nat.2)
    two_squared_is_four
    Nat.2.pow(Nat.2) = Nat.4
    nat_mul_2_4
    Nat.2 * Nat.4 = Nat.8
    Nat.2.pow(Nat.3) = Nat.8
}

/// The square of three: 3² = 9.
theorem three_squared_is_nine {
    Nat.3.pow(Nat.2) = Nat.9
} by {
    sq_eq_mul(Nat.3)
    Nat.3.pow(Nat.2) = Nat.3 * Nat.3
    nat_mul_3_3
    Nat.3 * Nat.3 = Nat.9
    Nat.3.pow(Nat.2) = Nat.9
}

/// Four is a perfect power: 4 = 2².
theorem four_is_perfect_power {
    is_perfect_power(Nat.4)
} by {
    two_squared_is_four
    Nat.2.pow(Nat.2) = Nat.4
    Nat.2 <= Nat.2
    is_perfect_power_intro(Nat.4, Nat.2, Nat.2)
}

/// Eight is a perfect power: 8 = 2³.
theorem eight_is_perfect_power {
    is_perfect_power(Nat.8)
} by {
    two_cubed_is_eight
    Nat.2.pow(Nat.3) = Nat.8
    Nat.2 <= Nat.3
    is_perfect_power_intro(Nat.8, Nat.2, Nat.3)
}

/// Nine is a perfect power: 9 = 3².
theorem nine_is_perfect_power {
    is_perfect_power(Nat.9)
} by {
    three_squared_is_nine
    Nat.3.pow(Nat.2) = Nat.9
    Nat.2 <= Nat.2
    is_perfect_power_intro(Nat.9, Nat.3, Nat.2)
}

// ============================================================================
// Section 2: the exponent law for products with a common base
// ============================================================================

/// The exponent law a^m · a^n = a^(m + n): the product of two powers with a
/// common base is the power with the summed exponent.
theorem perfect_power_product_law(a: Nat, m: Nat, n: Nat) {
    a.pow(m) * a.pow(n) = a.pow(m + n)
} by {
    pow_add[Nat](a, m, n)
}

/// The product of two perfect powers with a common base a, a^m and a^n, is
/// again a perfect power whenever m + n >= 2, since a^m · a^n = a^(m + n).
theorem product_of_perfect_powers_common_base(a: Nat, m: Nat, n: Nat) {
    Nat.2 <= m + n implies is_perfect_power(a.pow(m) * a.pow(n))
} by {
    if Nat.2 <= m + n {
        perfect_power_product_law(a, m, n)
        a.pow(m) * a.pow(n) = a.pow(m + n)
        is_perfect_power_intro(a.pow(m) * a.pow(n), a, m + n)
    }
}

// ============================================================================
// Section 3: two and six are not perfect powers
// ============================================================================

/// The multiplicity of a prime in a power: for a prime p and a nonzero base
/// a, the multiplicity of p in a^k is k times its multiplicity in a.  This is
/// the p-adic valuation identity v_p(a^k) = k · v_p(a).
theorem count_prime_factor_pow_mul(p: Nat, a: Nat, k: Nat) {
    p.is_prime and a != Nat.0 implies
        count_prime_factor(p, a.pow(k)) = k * count_prime_factor(p, a)
} by {
    if p.is_prime and a != Nat.0 {
        let f: Nat -> Bool = function(x: Nat) {
            count_prime_factor(p, a.pow(x)) = x * count_prime_factor(p, a)
        }
        a.pow(Nat.0) = Nat.1
        count_prime_factor_one(p)
        count_prime_factor(p, Nat.1) = Nat.0
        f(Nat.0)
        forall(x: Nat) {
            if f(x) {
                exp_ne_zero(a, x)
                a.pow(x) != Nat.0
                a.pow(x.suc) = a * a.pow(x)
                count_prime_factor_mul(p, a, a.pow(x))
                count_prime_factor(p, a * a.pow(x)) =
                    count_prime_factor(p, a) + count_prime_factor(p, a.pow(x))
                count_prime_factor(p, a.pow(x)) = x * count_prime_factor(p, a)
                count_prime_factor(p, a.pow(x.suc)) =
                    count_prime_factor(p, a) + x * count_prime_factor(p, a)
                x.suc * count_prime_factor(p, a) =
                    count_prime_factor(p, a) + x * count_prime_factor(p, a)
                count_prime_factor(p, a.pow(x.suc)) =
                    x.suc * count_prime_factor(p, a)
                f(x.suc)
            }
        }
        f(k)
        count_prime_factor(p, a.pow(k)) = k * count_prime_factor(p, a)
    }
}

/// If a product of naturals is one, the first factor is one: from k * c = 1,
/// k divides one, and the only divisor of one is one.
theorem mul_eq_one_imp_first_one(k: Nat, c: Nat) {
    k * c = Nat.1 implies k = Nat.1
} by {
    if k * c = Nat.1 {
        k.divides(Nat.1) = exists(w: Nat) { k * w = Nat.1 }
        k.divides(Nat.1)
        nat_divides_one_imp_one(k)
        k = Nat.1
    }
}

/// A base of a power equal to a nonzero number is nonzero: if a^k = n and
/// k >= 2, then a != 0, since zero to a positive power is zero.
theorem pow_base_nonzero_of_power(a: Nat, k: Nat, n: Nat) {
    Nat.2 <= k and n = a.pow(k) and n != Nat.0 implies a != Nat.0
} by {
    if Nat.2 <= k and n = a.pow(k) and n != Nat.0 {
        if a = Nat.0 {
            Nat.1 < Nat.2
            lt_and_lte(Nat.1, Nat.2, k)
            Nat.1 < k
            if k = Nat.0 {
                Nat.1 < Nat.0
                not_lt_zero(Nat.1)
                false
            }
            k != Nat.0
            zero_exp(k)
            Nat.0.pow(k) = Nat.0
            n = a.pow(k)
            a.pow(k) = Nat.0
            n = Nat.0
            false
        }
        a != Nat.0
    }
}

/// Two is not a perfect power: if 2 = a^k with k >= 2, then the prime two
/// occurs with multiplicity one in 2 but with multiplicity k · v_2(a), a
/// multiple of k >= 2, in a^k; a positive multiple of a number at least two
/// is never one, so k = 1, contradicting k >= 2.
theorem two_not_perfect_power {
    not is_perfect_power(Nat.2)
} by {
    if is_perfect_power(Nat.2) {
        is_perfect_power_apply(Nat.2)
        exists(a: Nat, k: Nat) {
            Nat.2 <= k and Nat.2 = a.pow(k)
        }
        let (a: Nat, k: Nat) satisfy { Nat.2 <= k and Nat.2 = a.pow(k) }
        pow_base_nonzero_of_power(a, k, Nat.2)
        a != Nat.0
        nat_two_prime
        Nat.2.is_prime
        count_prime_factor_pow_mul(Nat.2, a, k)
        count_prime_factor(Nat.2, a.pow(k)) = k * count_prime_factor(Nat.2, a)
        count_prime_factor_self(Nat.2)
        count_prime_factor(Nat.2, Nat.2) = Nat.1
        Nat.2 = a.pow(k)
        count_prime_factor(Nat.2, Nat.2) = count_prime_factor(Nat.2, a.pow(k))
        count_prime_factor(Nat.2, a.pow(k)) = Nat.1
        k * count_prime_factor(Nat.2, a) = Nat.1
        mul_eq_one_imp_first_one(k, count_prime_factor(Nat.2, a))
        k = Nat.1
        Nat.2 <= k
        Nat.2 <= Nat.1
        lte_imp_not_lt(Nat.2, Nat.1)
        Nat.1 < Nat.2
        false
    }
}

/// Six is not a perfect power: the prime two occurs with multiplicity one in
/// 6 = 2 · 3, but with multiplicity a multiple of k >= 2 in any k-th power.
theorem six_not_perfect_power {
    not is_perfect_power(Nat.6)
} by {
    if is_perfect_power(Nat.6) {
        is_perfect_power_apply(Nat.6)
        exists(a: Nat, k: Nat) {
            Nat.2 <= k and Nat.6 = a.pow(k)
        }
        let (a: Nat, k: Nat) satisfy { Nat.2 <= k and Nat.6 = a.pow(k) }
        pow_base_nonzero_of_power(a, k, Nat.6)
        a != Nat.0
        nat_two_prime
        Nat.2.is_prime
        three_is_prime
        Nat.3.is_prime
        count_prime_factor_pow_mul(Nat.2, a, k)
        count_prime_factor(Nat.2, a.pow(k)) = k * count_prime_factor(Nat.2, a)
        nat_mul_2_3
        Nat.2 * Nat.3 = Nat.6
        Nat.2 != Nat.0
        Nat.3 != Nat.0
        count_prime_factor_mul(Nat.2, Nat.2, Nat.3)
        count_prime_factor(Nat.2, Nat.2 * Nat.3) =
            count_prime_factor(Nat.2, Nat.2) + count_prime_factor(Nat.2, Nat.3)
        count_prime_factor_self(Nat.2)
        count_prime_factor(Nat.2, Nat.2) = Nat.1
        Nat.2 != Nat.3
        count_prime_factor_other_prime(Nat.2, Nat.3)
        count_prime_factor(Nat.2, Nat.3) = Nat.0
        count_prime_factor(Nat.2, Nat.2 * Nat.3) = Nat.1
        count_prime_factor(Nat.2, Nat.6) = Nat.1
        Nat.6 = a.pow(k)
        count_prime_factor(Nat.2, Nat.6) = count_prime_factor(Nat.2, a.pow(k))
        count_prime_factor(Nat.2, a.pow(k)) = Nat.1
        k * count_prime_factor(Nat.2, a) = Nat.1
        mul_eq_one_imp_first_one(k, count_prime_factor(Nat.2, a))
        k = Nat.1
        Nat.2 <= k
        Nat.2 <= Nat.1
        lte_imp_not_lt(Nat.2, Nat.1)
        Nat.1 < Nat.2
        false
    }
}

// ============================================================================
// Section 4: Catalan's conjecture (statement)
// ============================================================================

// Catalan's conjecture, proved by Mihăilescu in 2002: the only consecutive
// perfect powers are 8 = 2³ and 9 = 3².  Equivalently, the only solution of
// x^a - y^b = 1 in naturals with a, b >= 2 is (x, a, y, b) = (3, 2, 2, 3).
// The proof is deep — it runs through the arithmetic of cyclotomic fields and
// the Catalan equation — and is not formalized here.
//
// theorem catalan_consecutive_perfect_powers {
//     forall(x: Nat, a: Nat, y: Nat, b: Nat) {
//         Nat.2 <= a and Nat.2 <= b and x.pow(a) = y.pow(b) + Nat.1
//             implies (x = Nat.3 and a = Nat.2 and y = Nat.2 and b = Nat.3)
//     }
// }
