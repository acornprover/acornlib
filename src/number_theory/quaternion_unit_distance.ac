/// Quaternions as pairs of complex numbers: the unit-distance isometries.
///
/// This file packages the Cayley–Dickson presentation of the quaternions
/// (a quaternion is a pair (z, w) of complex numbers) with the norm
/// N(z, w) = |z|² + |w|² and the product
/// (z1, w1)·(z2, w2) = (z1z2 - w1·conj(w2), z1w2 + w1·conj(z2)) from
/// `number_theory/four_squares_identity.ac`, and derives the isometry
/// statements used by the 2026 disproof of the Erdős unit-distance
/// conjecture:
///
///   * the squared Euclidean distance of two quaternions is the norm of
///     their difference (`quat_dist_sq_eq_norm_sub`), and
///   * right multiplication by a unit quaternion preserves distances
///     (`quat_unit_dist_preserved`) and unit distances
///     (`quat_unit_dist_iff`).
///
/// It then formalizes the counting engine of the disproof: for a finite set
/// `s` of quaternions, `nu(s)` counts the unordered unit pairs, and the
/// count is invariant under multiplication by a unit quaternion
/// (`nu_unit_invariant`: nu(s·q) = nu(s) when N(q) = 1).  The proof is a
/// bijection: right multiplication by a unit quaternion is injective (its
/// inverse is right multiplication by the conjugate,
/// `quat_mul_right_inj`), and it preserves exactly the unit pairs, so the
/// ordered-unit-pair set of s·q is the image of that of s under an
/// injective map (`ordered_unit_pairs_unit_translate`).
///
/// The `quaternion_unit_distance` predicate is the unit-distance graph of
/// the four-dimensional integer lattice inside the quaternions.
from real import Real, from_nat_real_pos_of_ne_zero
from nat import Nat, from_nat, from_nat_one, from_nat_zero
from pair import Pair, pair_new_first, pair_new_second, pair_ext
from finite_set import FiniteSet, fs_image, finite_set_image_contains_eq, finite_set_ext_contains,
    finite_set_image_cardinality_is_of_injective
from data.basic.functions import is_injective_fn
from data.finite.finite_set_card import fs_card, fs_card_eq_of_cardinality_is, fs_card_cardinality_is
from data.finite.finite_set_filter import finite_set_filter, finite_set_filter_contains_eq,
    finite_set_filter_subset
from data.finite.finite_set_product import finite_set_product, finite_set_product_contains_eq,
    finite_set_product_contains_pair
from data.finite.finite_set_card_compare import fs_card_mono
from data.finite.finite_set_product_card import fs_card_product
from complex import Complex, distrib, abs_squared_conj, abs_squared_neg, abs_squared_eq,
    conj_conj, re_mul, im_mul, mul_comm, mul_assoc, mul_from_real, real_add_lifts, add_comm
from algebra.add_group import inverse_inverse, inverse_add, left_cancel, inverse_left
from algebra.ring.ring import mul_neg_left, mul_neg_right
from order import lt_not_ref, lte_trans_eq
from comm_ring import CommRing
from number_theory.four_squares_identity import norm2, qmul_z, qmul_w, norm2_mul, conj_neg2,
    conj_mul2, swap_pair, absorb_pair

numerals Real

/// A quaternion presented as a pair of complex numbers.
structure Quat {
    /// The first complex coordinate.
    first: Complex
    /// The second complex coordinate.
    second: Complex
}

/// The product of two quaternions.
define quat_mul(x: Quat, y: Quat) -> Quat {
    Quat.new(qmul_z(x.first, x.second, y.first, y.second),
        qmul_w(x.first, x.second, y.first, y.second))
}

/// The norm of a quaternion.
define quat_norm(x: Quat) -> Real {
    norm2(x.first, x.second)
}

/// The difference of two quaternions.
define quat_sub(x: Quat, y: Quat) -> Quat {
    Quat.new(x.first - y.first, x.second - y.second)
}

/// The squared Euclidean distance of two quaternions.
define quat_dist_sq(x: Quat, y: Quat) -> Real {
    norm2(x.first - y.first, x.second - y.second)
}

/// True when two quaternions are at Euclidean distance exactly one.
define quaternion_unit_distance(x: Quat, y: Quat) -> Bool {
    quat_dist_sq(x, y) = Real.1
}

/// The squared distance of two quaternions is the norm of their difference.
theorem quat_dist_sq_eq_norm_sub(x: Quat, y: Quat) {
    quat_dist_sq(x, y) = quat_norm(quat_sub(x, y))
} by {
    quat_dist_sq(x, y) = norm2(x.first - y.first, x.second - y.second)
    quat_norm(quat_sub(x, y)) = norm2(quat_sub(x, y).first, quat_sub(x, y).second)
    quat_sub(x, y) = Quat.new(x.first - y.first, x.second - y.second)
    quat_sub(x, y).first = x.first - y.first
    quat_sub(x, y).second = x.second - y.second
    quat_norm(quat_sub(x, y)) = norm2(x.first - y.first, x.second - y.second)
    quat_dist_sq(x, y) = quat_norm(quat_sub(x, y))
}

/// Multiplication distributes over subtraction: (a - b)·c = a·c - b·c.
theorem mul_sub_distrib(a: Complex, b: Complex, c: Complex) {
    (a - b) * c = a * c - b * c
} by {
    distrib(a, -b, c)
    (a + -b) * c = a * c + -b * c
    mul_neg_left(b, c)
    -b * c = -(b * c)
    (a + -b) * c = a * c + -(b * c)
    (a - b) * c = a * c - b * c
}

/// Reorder four negated summands: x + -y + -p + q = x + -p + -y + q.
theorem cr_re4[R: CommRing](a: R, b: R, c: R, d: R) {
    a + -b + -c + d = a + -c + -b + d
} by {
}

/// (X - Y) - (P - Q) = (X - P) - (Y - Q).
theorem sub_sub_rearrange[R: CommRing](x: R, y: R, p: R, q: R) {
    (x - y) - (p - q) = (x - p) - (y - q)
} by {
    (x - y) - (p - q) = (x + -y) + -(p + -q)
    -(p + -q) = -p + -(-q)
    -(-q) = q
    (x + -y) + -(p + -q) = (x + -y) + (-p + q)
    (x + -y) + (-p + q) = x + -y + -p + q
    cr_re4(x, y, p, q)
    x + -y + -p + q = x + -p + -y + q
    x + -p + -y + q = (x - p) + (-y + q)
    -(y + -q) = -y + q
    (x - p) + (-y + q) = (x - p) + -(y + -q)
    (x - p) + -(y + -q) = (x - p) - (y - q)
}

/// (X - Y) + (P - Q) = (X + P) - (Y + Q).
theorem sub_add_rearrange[R: CommRing](x: R, y: R, p: R, q: R) {
    (x - y) + (p - q) = (x + p) - (y + q)
} by {
    (x - y) + (p - q) = (x + -y) + (p + -q)
    cr_re4(x, p, y, q)
    x + -y + -p + q = x + -p + -y + q
    (x + -y) + (p + -q) = x + -y + p + -q
    x + -y + p + -q = x + p + -y + -q
    -(y + q) = -y + -q
    x + p + -y + -q = (x + p) + -(y + q)
    (x + p) + -(y + q) = (x + p) - (y + q)
}

/// The norm of a product is the product of the norms.
theorem quat_norm_mul(x: Quat, y: Quat) {
    quat_norm(quat_mul(x, y)) = quat_norm(x) * quat_norm(y)
} by {
    quat_norm(quat_mul(x, y)) = norm2(quat_mul(x, y).first, quat_mul(x, y).second)
    quat_mul(x, y) = Quat.new(qmul_z(x.first, x.second, y.first, y.second),
        qmul_w(x.first, x.second, y.first, y.second))
    quat_mul(x, y).first = qmul_z(x.first, x.second, y.first, y.second)
    quat_mul(x, y).second = qmul_w(x.first, x.second, y.first, y.second)
    quat_norm(quat_mul(x, y)) =
        norm2(qmul_z(x.first, x.second, y.first, y.second),
            qmul_w(x.first, x.second, y.first, y.second))
    quat_norm(x) = norm2(x.first, x.second)
    quat_norm(y) = norm2(y.first, y.second)
    quat_norm(x) * quat_norm(y) =
        norm2(x.first, x.second) * norm2(y.first, y.second)
    norm2_mul(x.first, x.second, y.first, y.second)
    norm2(x.first, x.second) * norm2(y.first, y.second) =
        norm2(qmul_z(x.first, x.second, y.first, y.second),
            qmul_w(x.first, x.second, y.first, y.second))
    quat_norm(quat_mul(x, y)) = quat_norm(x) * quat_norm(y)
}

/// The first coordinate of the product distributes over subtraction.
theorem qmul_z_sub(z1: Complex, z2: Complex, w1: Complex, w2: Complex, z3: Complex, w3: Complex) {
    qmul_z(z1 - z2, w1 - w2, z3, w3) =
        qmul_z(z1, w1, z3, w3) - qmul_z(z2, w2, z3, w3)
} by {
    qmul_z(z1 - z2, w1 - w2, z3, w3) = (z1 - z2) * z3 - (w1 - w2) * w3.conj
    mul_sub_distrib(z1, z2, z3)
    (z1 - z2) * z3 = z1 * z3 - z2 * z3
    mul_sub_distrib(w1, w2, w3.conj)
    (w1 - w2) * w3.conj = w1 * w3.conj - w2 * w3.conj
    (z1 - z2) * z3 - (w1 - w2) * w3.conj = z1 * z3 - z2 * z3 - (w1 * w3.conj - w2 * w3.conj)
    sub_sub_rearrange(z1 * z3, z2 * z3, w1 * w3.conj, w2 * w3.conj)
    z1 * z3 - z2 * z3 - (w1 * w3.conj - w2 * w3.conj) =
        z1 * z3 - w1 * w3.conj - (z2 * z3 - w2 * w3.conj)
    qmul_z(z1 - z2, w1 - w2, z3, w3) =
        z1 * z3 - w1 * w3.conj - (z2 * z3 - w2 * w3.conj)
    qmul_z(z1, w1, z3, w3) = z1 * z3 - w1 * w3.conj
    qmul_z(z2, w2, z3, w3) = z2 * z3 - w2 * w3.conj
    qmul_z(z1 - z2, w1 - w2, z3, w3) =
        qmul_z(z1, w1, z3, w3) - qmul_z(z2, w2, z3, w3)
}
/// The second coordinate of the product distributes over subtraction.
theorem qmul_w_sub(z1: Complex, z2: Complex, w1: Complex, w2: Complex, z3: Complex, w3: Complex) {
    qmul_w(z1 - z2, w1 - w2, z3, w3) =
        qmul_w(z1, w1, z3, w3) - qmul_w(z2, w2, z3, w3)
} by {
    qmul_w(z1 - z2, w1 - w2, z3, w3) = (z1 - z2) * w3 + (w1 - w2) * z3.conj
    mul_sub_distrib(z1, z2, w3)
    (z1 - z2) * w3 = z1 * w3 - z2 * w3
    mul_sub_distrib(w1, w2, z3.conj)
    (w1 - w2) * z3.conj = w1 * z3.conj - w2 * z3.conj
    (z1 - z2) * w3 + (w1 - w2) * z3.conj = z1 * w3 - z2 * w3 + (w1 * z3.conj - w2 * z3.conj)
    sub_add_rearrange(z1 * w3, z2 * w3, w1 * z3.conj, w2 * z3.conj)
    z1 * w3 - z2 * w3 + (w1 * z3.conj - w2 * z3.conj) =
        z1 * w3 + w1 * z3.conj - (z2 * w3 + w2 * z3.conj)
    qmul_w(z1 - z2, w1 - w2, z3, w3) =
        z1 * w3 + w1 * z3.conj - (z2 * w3 + w2 * z3.conj)
    qmul_w(z1, w1, z3, w3) = z1 * w3 + w1 * z3.conj
    qmul_w(z2, w2, z3, w3) = z2 * w3 + w2 * z3.conj
    qmul_w(z1 - z2, w1 - w2, z3, w3) =
        qmul_w(z1, w1, z3, w3) - qmul_w(z2, w2, z3, w3)
}
/// Multiplication distributes over subtraction: (x - y)·q = x·q - y·q.
theorem quat_mul_dist_sub(x: Quat, y: Quat, q: Quat) {
    quat_mul(quat_sub(x, y), q) = quat_sub(quat_mul(x, q), quat_mul(y, q))
} by {
    quat_mul(quat_sub(x, y), q) =
        Quat.new(qmul_z(quat_sub(x, y).first, quat_sub(x, y).second, q.first, q.second),
            qmul_w(quat_sub(x, y).first, quat_sub(x, y).second, q.first, q.second))
    quat_sub(x, y).first = x.first - y.first
    quat_sub(x, y).second = x.second - y.second
    quat_mul(quat_sub(x, y), q) =
        Quat.new(qmul_z(x.first - y.first, x.second - y.second, q.first, q.second),
            qmul_w(x.first - y.first, x.second - y.second, q.first, q.second))
    qmul_z_sub(x.first, y.first, x.second, y.second, q.first, q.second)
    qmul_z(x.first - y.first, x.second - y.second, q.first, q.second) =
        qmul_z(x.first, x.second, q.first, q.second) -
        qmul_z(y.first, y.second, q.first, q.second)
    qmul_w_sub(x.first, y.first, x.second, y.second, q.first, q.second)
    qmul_w(x.first - y.first, x.second - y.second, q.first, q.second) =
        qmul_w(x.first, x.second, q.first, q.second) -
        qmul_w(y.first, y.second, q.first, q.second)
    Quat.new(qmul_z(x.first - y.first, x.second - y.second, q.first, q.second),
        qmul_w(x.first - y.first, x.second - y.second, q.first, q.second)) =
        Quat.new(qmul_z(x.first, x.second, q.first, q.second) -
            qmul_z(y.first, y.second, q.first, q.second),
            qmul_w(x.first, x.second, q.first, q.second) -
            qmul_w(y.first, y.second, q.first, q.second))
    quat_sub(quat_mul(x, q), quat_mul(y, q)) =
        Quat.new(quat_mul(x, q).first - quat_mul(y, q).first,
            quat_mul(x, q).second - quat_mul(y, q).second)
    quat_mul(x, q).first = qmul_z(x.first, x.second, q.first, q.second)
    quat_mul(y, q).first = qmul_z(y.first, y.second, q.first, q.second)
    quat_mul(x, q).second = qmul_w(x.first, x.second, q.first, q.second)
    quat_mul(y, q).second = qmul_w(y.first, y.second, q.first, q.second)
    quat_sub(quat_mul(x, q), quat_mul(y, q)) =
        Quat.new(qmul_z(x.first, x.second, q.first, q.second) -
            qmul_z(y.first, y.second, q.first, q.second),
            qmul_w(x.first, x.second, q.first, q.second) -
            qmul_w(y.first, y.second, q.first, q.second))
    quat_mul(quat_sub(x, y), q) = quat_sub(quat_mul(x, q), quat_mul(y, q))
}



/// Right multiplication by a unit quaternion preserves all distances:
/// if N(q) = 1 then dist(x·q, y·q) = dist(x, y).
theorem quat_unit_dist_preserved(x: Quat, y: Quat, q: Quat) {
    quat_norm(q) = Real.1 implies
        quat_dist_sq(quat_mul(x, q), quat_mul(y, q)) = quat_dist_sq(x, y)
} by {
    if quat_norm(q) = Real.1 {
        quat_dist_sq(quat_mul(x, q), quat_mul(y, q)) =
            quat_norm(quat_sub(quat_mul(x, q), quat_mul(y, q)))
        quat_mul_dist_sub(x, y, q)
        quat_mul(quat_sub(x, y), q) = quat_sub(quat_mul(x, q), quat_mul(y, q))
        quat_dist_sq(quat_mul(x, q), quat_mul(y, q)) =
            quat_norm(quat_mul(quat_sub(x, y), q))
        quat_norm_mul(quat_sub(x, y), q)
        quat_norm(quat_mul(quat_sub(x, y), q)) =
            quat_norm(quat_sub(x, y)) * quat_norm(q)
        quat_dist_sq(quat_mul(x, q), quat_mul(y, q)) =
            quat_norm(quat_sub(x, y)) * quat_norm(q)
        quat_dist_sq(x, y) = quat_norm(quat_sub(x, y))
        quat_dist_sq(quat_mul(x, q), quat_mul(y, q)) =
            quat_dist_sq(x, y) * quat_norm(q)
        quat_dist_sq(quat_mul(x, q), quat_mul(y, q)) = quat_dist_sq(x, y) * Real.1
        quat_dist_sq(quat_mul(x, q), quat_mul(y, q)) = quat_dist_sq(x, y)
    }
}

/// A unit quaternion maps unit distances to unit distances, in both
/// directions: x and y are at distance one exactly when x·q and y·q are.
theorem quat_unit_dist_iff(x: Quat, y: Quat, q: Quat) {
    quat_norm(q) = Real.1 implies
        (quaternion_unit_distance(x, y) = quaternion_unit_distance(quat_mul(x, q), quat_mul(y, q)))
} by {
    if quat_norm(q) = Real.1 {
        quat_unit_dist_preserved(x, y, q)
        quat_dist_sq(quat_mul(x, q), quat_mul(y, q)) = quat_dist_sq(x, y)
        if quaternion_unit_distance(x, y) {
            quat_dist_sq(x, y) = Real.1
            quat_dist_sq(quat_mul(x, q), quat_mul(y, q)) = Real.1
            quaternion_unit_distance(quat_mul(x, q), quat_mul(y, q))
        }
        quaternion_unit_distance(x, y) implies quaternion_unit_distance(quat_mul(x, q), quat_mul(y, q))
        quat_unit_dist_preserved(quat_mul(x, q), quat_mul(y, q), q)
        quat_dist_sq(quat_mul(quat_mul(x, q), q), quat_mul(quat_mul(y, q), q)) =
            quat_dist_sq(quat_mul(x, q), quat_mul(y, q))
        if quaternion_unit_distance(quat_mul(x, q), quat_mul(y, q)) {
            quat_unit_dist_preserved(x, y, q)
            quat_dist_sq(quat_mul(x, q), quat_mul(y, q)) = quat_dist_sq(x, y)
            quat_dist_sq(x, y) = Real.1
            quaternion_unit_distance(x, y)
        }
        quaternion_unit_distance(quat_mul(x, q), quat_mul(y, q)) implies quaternion_unit_distance(x, y)
        quaternion_unit_distance(x, y) = quaternion_unit_distance(quat_mul(x, q), quat_mul(y, q))
    }
}

// ============================================================================
// The conjugate and the bijection by a unit quaternion
// ============================================================================
//
// Right multiplication by a unit quaternion is a bijection of the
// quaternions: its inverse is right multiplication by the conjugate, because
// (x·q)·conj(q) = x·N(q) and N(q) = 1 for unit q.  This is the map under
// which the unit-distance count of a finite set is invariant.

/// The conjugate of a quaternion.
define quat_conj(x: Quat) -> Quat {
    Quat.new(x.first.conj, -x.second)
}

/// The first coordinate of q·conj(q) is the norm.
theorem quat_mul_conj_first(x: Quat) {
    qmul_z(x.first, x.second, x.first.conj, -(x.second)) =
        Complex.from_real(quat_norm(x))
} by {
    qmul_z(x.first, x.second, x.first.conj, -(x.second)) =
        x.first * x.first.conj - x.second * (-(x.second)).conj
    conj_neg2(x.second)
    (-(x.second)).conj = -(x.second.conj)
    x.second * (-(x.second)).conj = x.second * (-(x.second.conj))
    mul_neg_right(x.second, x.second.conj)
    x.second * (-(x.second.conj)) = -(x.second * x.second.conj)
    x.second * (-(x.second)).conj = -(x.second * x.second.conj)
    x.first * x.first.conj - x.second * (-(x.second)).conj =
        x.first * x.first.conj - (-(x.second * x.second.conj))
    x.first * x.first.conj - (-(x.second * x.second.conj)) =
        x.first * x.first.conj + x.second * x.second.conj
    abs_squared_conj(x.first)
    x.first * x.first.conj = Complex.new(x.first.abs_squared, Real.0)
    abs_squared_conj(x.second)
    x.second * x.second.conj = Complex.new(x.second.abs_squared, Real.0)
    x.first * x.first.conj + x.second * x.second.conj =
        Complex.new(x.first.abs_squared + x.second.abs_squared, Real.0)
    quat_norm(x) = norm2(x.first, x.second)
    norm2(x.first, x.second) = x.first.abs_squared + x.second.abs_squared
    Complex.from_real(quat_norm(x)) =
        Complex.new(x.first.abs_squared + x.second.abs_squared, Real.0)
    x.first * x.first.conj + x.second * x.second.conj = Complex.from_real(quat_norm(x))
    qmul_z(x.first, x.second, x.first.conj, -(x.second)) = Complex.from_real(quat_norm(x))
}

/// The second coordinate of q·conj(q) is zero.
theorem quat_mul_conj_second(x: Quat) {
    qmul_w(x.first, x.second, x.first.conj, -(x.second)) = Complex.0
} by {
    qmul_w(x.first, x.second, x.first.conj, -(x.second)) =
        x.first * (-(x.second)) + x.second * x.first.conj.conj
    conj_conj(x.first)
    x.first.conj.conj = x.first
    x.first * (-(x.second)) + x.second * x.first.conj.conj =
        x.first * (-(x.second)) + x.second * x.first
    mul_neg_right(x.first, x.second)
    x.first * (-(x.second)) = -(x.first * x.second)
    x.second * x.first = x.first * x.second
    x.first * (-(x.second)) + x.second * x.first =
        -(x.first * x.second) + x.first * x.second
    -(x.first * x.second) + x.first * x.second = Complex.0
    x.first * (-(x.second)) + x.second * x.first = Complex.0
    qmul_w(x.first, x.second, x.first.conj, -(x.second)) = Complex.0
}

/// q·conj(q) is the pair (N(q), 0): the norm embedded as the first coordinate.
theorem quat_mul_conj_self(x: Quat) {
    quat_mul(x, quat_conj(x)) = Quat.new(Complex.from_real(quat_norm(x)), Complex.0)
} by {
    quat_mul(x, quat_conj(x)) =
        Quat.new(qmul_z(x.first, x.second, quat_conj(x).first, quat_conj(x).second),
            qmul_w(x.first, x.second, quat_conj(x).first, quat_conj(x).second))
    quat_conj(x) = Quat.new(x.first.conj, -(x.second))
    quat_conj(x).first = x.first.conj
    quat_conj(x).second = -(x.second)
    quat_mul(x, quat_conj(x)) =
        Quat.new(qmul_z(x.first, x.second, x.first.conj, -(x.second)),
            qmul_w(x.first, x.second, x.first.conj, -(x.second)))
    quat_mul_conj_first(x)
    qmul_z(x.first, x.second, x.first.conj, -(x.second)) =
        Complex.from_real(quat_norm(x))
    quat_mul_conj_second(x)
    qmul_w(x.first, x.second, x.first.conj, -(x.second)) = Complex.0
    quat_mul(x, quat_conj(x)) = Quat.new(Complex.from_real(quat_norm(x)), Complex.0)
}

/// The first coordinate of (x·q)·conj(q) is x.first scaled by the norm of q.
theorem quat_mul_right_conj_first(x: Quat, q: Quat) {
    qmul_z(qmul_z(x.first, x.second, q.first, q.second),
        qmul_w(x.first, x.second, q.first, q.second),
        q.first.conj, -(q.second)) =
        x.first * Complex.from_real(quat_norm(q))
} by {
    qmul_z(qmul_z(x.first, x.second, q.first, q.second),
        qmul_w(x.first, x.second, q.first, q.second),
        q.first.conj, -(q.second)) =
        qmul_z(x.first, x.second, q.first, q.second) * q.first.conj -
        qmul_w(x.first, x.second, q.first, q.second) * (-(q.second)).conj
    conj_neg2(q.second)
    (-(q.second)).conj = -(q.second.conj)
    qmul_z(x.first, x.second, q.first, q.second) = x.first * q.first - x.second * q.second.conj
    qmul_w(x.first, x.second, q.first, q.second) = x.first * q.second + x.second * q.first.conj
    qmul_z(x.first, x.second, q.first, q.second) * q.first.conj -
        qmul_w(x.first, x.second, q.first, q.second) * (-(q.second)).conj =
        (x.first * q.first - x.second * q.second.conj) * q.first.conj -
        (x.first * q.second + x.second * q.first.conj) * (-(q.second.conj))
    mul_neg_right(x.first * q.second + x.second * q.first.conj, q.second.conj)
    (x.first * q.second + x.second * q.first.conj) * (-(q.second.conj)) =
        -((x.first * q.second + x.second * q.first.conj) * q.second.conj)
    (x.first * q.first - x.second * q.second.conj) * q.first.conj -
        (x.first * q.second + x.second * q.first.conj) * (-(q.second.conj)) =
        (x.first * q.first - x.second * q.second.conj) * q.first.conj -
        (-((x.first * q.second + x.second * q.first.conj) * q.second.conj))
    (x.first * q.first - x.second * q.second.conj) * q.first.conj -
        (-((x.first * q.second + x.second * q.first.conj) * q.second.conj)) =
        (x.first * q.first - x.second * q.second.conj) * q.first.conj +
        -(-((x.first * q.second + x.second * q.first.conj) * q.second.conj))
    inverse_inverse((x.first * q.second + x.second * q.first.conj) * q.second.conj)
    -(-((x.first * q.second + x.second * q.first.conj) * q.second.conj)) =
        (x.first * q.second + x.second * q.first.conj) * q.second.conj
    (x.first * q.first - x.second * q.second.conj) * q.first.conj +
        -(-((x.first * q.second + x.second * q.first.conj) * q.second.conj)) =
        (x.first * q.first - x.second * q.second.conj) * q.first.conj +
        (x.first * q.second + x.second * q.first.conj) * q.second.conj
    mul_sub_distrib(x.first * q.first, x.second * q.second.conj, q.first.conj)
    (x.first * q.first - x.second * q.second.conj) * q.first.conj =
        x.first * q.first * q.first.conj - x.second * q.second.conj * q.first.conj
    distrib(x.first * q.second, x.second * q.first.conj, q.second.conj)
    (x.first * q.second + x.second * q.first.conj) * q.second.conj =
        x.first * q.second * q.second.conj + x.second * q.first.conj * q.second.conj
    (x.first * q.first - x.second * q.second.conj) * q.first.conj +
        (x.first * q.second + x.second * q.first.conj) * q.second.conj =
        x.first * q.first * q.first.conj - x.second * q.second.conj * q.first.conj +
        x.first * q.second * q.second.conj + x.second * q.first.conj * q.second.conj
    x.first * q.first * q.first.conj - x.second * q.second.conj * q.first.conj +
        x.first * q.second * q.second.conj + x.second * q.first.conj * q.second.conj =
        x.first * (q.first * q.first.conj) - x.second * (q.second.conj * q.first.conj) +
        x.first * (q.second * q.second.conj) + x.second * (q.first.conj * q.second.conj)
    mul_comm(q.second.conj, q.first.conj)
    q.second.conj * q.first.conj = q.first.conj * q.second.conj
    x.second * (q.second.conj * q.first.conj) = x.second * (q.first.conj * q.second.conj)
    x.first * (q.first * q.first.conj) - x.second * (q.second.conj * q.first.conj) +
        x.first * (q.second * q.second.conj) + x.second * (q.first.conj * q.second.conj) =
        x.first * (q.first * q.first.conj) - x.second * (q.first.conj * q.second.conj) +
        x.first * (q.second * q.second.conj) + x.second * (q.first.conj * q.second.conj)
    swap_pair(x.first * (q.first * q.first.conj),
        -(x.second * (q.first.conj * q.second.conj)),
        x.first * (q.second * q.second.conj),
        x.second * (q.first.conj * q.second.conj))
    x.first * (q.first * q.first.conj) - x.second * (q.first.conj * q.second.conj) +
        x.first * (q.second * q.second.conj) + x.second * (q.first.conj * q.second.conj) =
        x.first * (q.first * q.first.conj) + x.first * (q.second * q.second.conj) +
        (-(x.second * (q.first.conj * q.second.conj))) +
        x.second * (q.first.conj * q.second.conj)
    absorb_pair(x.first * (q.first * q.first.conj) + x.first * (q.second * q.second.conj),
        -(x.second * (q.first.conj * q.second.conj)),
        -(x.second * (q.first.conj * q.second.conj)), Complex.0)
    x.first * (q.first * q.first.conj) + x.first * (q.second * q.second.conj) +
        (-(x.second * (q.first.conj * q.second.conj))) +
        -(-(x.second * (q.first.conj * q.second.conj))) + Complex.0 =
        x.first * (q.first * q.first.conj) + x.first * (q.second * q.second.conj) + Complex.0
    inverse_inverse(x.second * (q.first.conj * q.second.conj))
    -(-(x.second * (q.first.conj * q.second.conj))) =
        x.second * (q.first.conj * q.second.conj)
    x.first * (q.first * q.first.conj) + x.first * (q.second * q.second.conj) +
        (-(x.second * (q.first.conj * q.second.conj))) +
        -(-(x.second * (q.first.conj * q.second.conj))) + Complex.0 =
        x.first * (q.first * q.first.conj) + x.first * (q.second * q.second.conj) +
        (-(x.second * (q.first.conj * q.second.conj))) +
        x.second * (q.first.conj * q.second.conj) + Complex.0
    x.first * (q.first * q.first.conj) + x.first * (q.second * q.second.conj) +
        (-(x.second * (q.first.conj * q.second.conj))) +
        x.second * (q.first.conj * q.second.conj) + Complex.0 =
        x.first * (q.first * q.first.conj) + x.first * (q.second * q.second.conj)
    abs_squared_conj(q.first)
    q.first * q.first.conj = Complex.new(q.first.abs_squared, Real.0)
    abs_squared_conj(q.second)
    q.second * q.second.conj = Complex.new(q.second.abs_squared, Real.0)
    x.first * (q.first * q.first.conj) + x.first * (q.second * q.second.conj) =
        x.first * Complex.from_real(q.first.abs_squared) +
        x.first * Complex.from_real(q.second.abs_squared)
    mul_from_real(x.first, q.first.abs_squared)
    x.first * Complex.from_real(q.first.abs_squared) =
        Complex.new(x.first.re * q.first.abs_squared, x.first.im * q.first.abs_squared)
    mul_from_real(x.first, q.second.abs_squared)
    x.first * Complex.from_real(q.second.abs_squared) =
        Complex.new(x.first.re * q.second.abs_squared, x.first.im * q.second.abs_squared)
    x.first * Complex.from_real(q.first.abs_squared) +
        x.first * Complex.from_real(q.second.abs_squared) =
        Complex.new(x.first.re * q.first.abs_squared, x.first.im * q.first.abs_squared) +
        Complex.new(x.first.re * q.second.abs_squared, x.first.im * q.second.abs_squared)
    Complex.new(x.first.re * q.first.abs_squared, x.first.im * q.first.abs_squared) +
        Complex.new(x.first.re * q.second.abs_squared, x.first.im * q.second.abs_squared) =
        Complex.new(x.first.re * q.first.abs_squared + x.first.re * q.second.abs_squared,
            x.first.im * q.first.abs_squared + x.first.im * q.second.abs_squared)
    Complex.new(x.first.re * q.first.abs_squared + x.first.re * q.second.abs_squared,
        x.first.im * q.first.abs_squared + x.first.im * q.second.abs_squared) =
        Complex.new(x.first.re * (q.first.abs_squared + q.second.abs_squared),
            x.first.im * (q.first.abs_squared + q.second.abs_squared))
    mul_from_real(x.first, q.first.abs_squared + q.second.abs_squared)
    x.first * Complex.from_real(q.first.abs_squared + q.second.abs_squared) =
        Complex.new(x.first.re * (q.first.abs_squared + q.second.abs_squared),
            x.first.im * (q.first.abs_squared + q.second.abs_squared))
    x.first * Complex.from_real(q.first.abs_squared) +
        x.first * Complex.from_real(q.second.abs_squared) =
        x.first * Complex.from_real(q.first.abs_squared + q.second.abs_squared)
    quat_norm(q) = norm2(q.first, q.second)
    norm2(q.first, q.second) = q.first.abs_squared + q.second.abs_squared
    Complex.from_real(q.first.abs_squared + q.second.abs_squared) =
        Complex.from_real(quat_norm(q))
    x.first * Complex.from_real(q.first.abs_squared + q.second.abs_squared) =
        x.first * Complex.from_real(quat_norm(q))
    x.first * (q.first * q.first.conj) + x.first * (q.second * q.second.conj) =
        x.first * Complex.from_real(quat_norm(q))
    qmul_z(qmul_z(x.first, x.second, q.first, q.second),
        qmul_w(x.first, x.second, q.first, q.second),
        q.first.conj, -(q.second)) =
        x.first * Complex.from_real(quat_norm(q))
}

/// The second coordinate of (x·q)·conj(q) is x.second scaled by the norm of q.
theorem quat_mul_right_conj_second(x: Quat, q: Quat) {
    qmul_w(qmul_z(x.first, x.second, q.first, q.second),
        qmul_w(x.first, x.second, q.first, q.second),
        q.first.conj, -(q.second)) =
        x.second * Complex.from_real(quat_norm(q))
} by {
    qmul_w(qmul_z(x.first, x.second, q.first, q.second),
        qmul_w(x.first, x.second, q.first, q.second),
        q.first.conj, -(q.second)) =
        qmul_z(x.first, x.second, q.first, q.second) * (-(q.second)) +
        qmul_w(x.first, x.second, q.first, q.second) * q.first.conj.conj
    conj_conj(q.first)
    q.first.conj.conj = q.first
    qmul_z(x.first, x.second, q.first, q.second) = x.first * q.first - x.second * q.second.conj
    qmul_w(x.first, x.second, q.first, q.second) = x.first * q.second + x.second * q.first.conj
    qmul_z(x.first, x.second, q.first, q.second) * (-(q.second)) +
        qmul_w(x.first, x.second, q.first, q.second) * q.first.conj.conj =
        (x.first * q.first - x.second * q.second.conj) * (-(q.second)) +
        (x.first * q.second + x.second * q.first.conj) * q.first
    mul_neg_right(x.first * q.first - x.second * q.second.conj, q.second)
    (x.first * q.first - x.second * q.second.conj) * (-(q.second)) =
        -((x.first * q.first - x.second * q.second.conj) * q.second)
    (x.first * q.first - x.second * q.second.conj) * (-(q.second)) +
        (x.first * q.second + x.second * q.first.conj) * q.first =
        -((x.first * q.first - x.second * q.second.conj) * q.second) +
        (x.first * q.second + x.second * q.first.conj) * q.first
    mul_sub_distrib(x.first * q.first, x.second * q.second.conj, q.second)
    (x.first * q.first - x.second * q.second.conj) * q.second =
        x.first * q.first * q.second - x.second * q.second.conj * q.second
    -((x.first * q.first - x.second * q.second.conj) * q.second) +
        (x.first * q.second + x.second * q.first.conj) * q.first =
        -(x.first * q.first * q.second - x.second * q.second.conj * q.second) +
        (x.first * q.second + x.second * q.first.conj) * q.first
    inverse_add(x.first * q.first * q.second, -(x.second * q.second.conj * q.second))
    -(x.first * q.first * q.second + -(x.second * q.second.conj * q.second)) =
        -(x.first * q.first * q.second) + -(-(x.second * q.second.conj * q.second))
    inverse_inverse(x.second * q.second.conj * q.second)
    -(-(x.second * q.second.conj * q.second)) = x.second * q.second.conj * q.second
    -(x.first * q.first * q.second + -(x.second * q.second.conj * q.second)) =
        -(x.first * q.first * q.second) + x.second * q.second.conj * q.second
    x.first * q.first * q.second - x.second * q.second.conj * q.second =
        x.first * q.first * q.second + -(x.second * q.second.conj * q.second)
    -(x.first * q.first * q.second - x.second * q.second.conj * q.second) =
        -(x.first * q.first * q.second) + x.second * q.second.conj * q.second
    -(x.first * q.first * q.second - x.second * q.second.conj * q.second) +
        (x.first * q.second + x.second * q.first.conj) * q.first =
        -(x.first * q.first * q.second) + x.second * q.second.conj * q.second +
        (x.first * q.second + x.second * q.first.conj) * q.first
    distrib(x.first * q.second, x.second * q.first.conj, q.first)
    (x.first * q.second + x.second * q.first.conj) * q.first =
        x.first * q.second * q.first + x.second * q.first.conj * q.first
    -(x.first * q.first * q.second) + x.second * q.second.conj * q.second +
        (x.first * q.second + x.second * q.first.conj) * q.first =
        -(x.first * q.first * q.second) + x.second * q.second.conj * q.second +
        x.first * q.second * q.first + x.second * q.first.conj * q.first
    swap_pair(-(x.first * q.first * q.second),
        x.second * q.second.conj * q.second,
        x.first * q.second * q.first,
        x.second * q.first.conj * q.first)
    -(x.first * q.first * q.second) + x.second * q.second.conj * q.second +
        x.first * q.second * q.first + x.second * q.first.conj * q.first =
        -(x.first * q.first * q.second) + x.first * q.second * q.first +
        x.second * q.second.conj * q.second + x.second * q.first.conj * q.first
    mul_assoc(x.first, q.first, q.second)
    x.first * (q.first * q.second) = (x.first * q.first) * q.second
    mul_assoc(x.first, q.second, q.first)
    x.first * (q.second * q.first) = (x.first * q.second) * q.first
    mul_comm(q.first, q.second)
    q.first * q.second = q.second * q.first
    x.first * (q.first * q.second) = x.first * (q.second * q.first)
    x.first * q.first * q.second = x.first * q.second * q.first
    -(x.first * q.first * q.second) + x.first * q.second * q.first =
        -(x.first * q.first * q.second) + x.first * q.first * q.second
    -(x.first * q.first * q.second) + x.first * q.first * q.second = Complex.0
    -(x.first * q.first * q.second) + x.first * q.second * q.first +
        x.second * q.second.conj * q.second + x.second * q.first.conj * q.first =
        x.second * q.second.conj * q.second + x.second * q.first.conj * q.first
    abs_squared_conj(q.second)
    q.second * q.second.conj = Complex.new(q.second.abs_squared, Real.0)
    q.second.conj * q.second = q.second * q.second.conj
    abs_squared_conj(q.first)
    q.first * q.first.conj = Complex.new(q.first.abs_squared, Real.0)
    q.first.conj * q.first = q.first * q.first.conj
    x.second * q.second.conj * q.second + x.second * q.first.conj * q.first =
        x.second * Complex.from_real(q.second.abs_squared) +
        x.second * Complex.from_real(q.first.abs_squared)
    mul_from_real(x.second, q.second.abs_squared)
    x.second * Complex.from_real(q.second.abs_squared) =
        Complex.new(x.second.re * q.second.abs_squared, x.second.im * q.second.abs_squared)
    mul_from_real(x.second, q.first.abs_squared)
    x.second * Complex.from_real(q.first.abs_squared) =
        Complex.new(x.second.re * q.first.abs_squared, x.second.im * q.first.abs_squared)
    x.second * Complex.from_real(q.second.abs_squared) +
        x.second * Complex.from_real(q.first.abs_squared) =
        Complex.new(x.second.re * q.second.abs_squared, x.second.im * q.second.abs_squared) +
        Complex.new(x.second.re * q.first.abs_squared, x.second.im * q.first.abs_squared)
    Complex.new(x.second.re * q.second.abs_squared, x.second.im * q.second.abs_squared) +
        Complex.new(x.second.re * q.first.abs_squared, x.second.im * q.first.abs_squared) =
        Complex.new(x.second.re * (q.second.abs_squared + q.first.abs_squared),
            x.second.im * (q.second.abs_squared + q.first.abs_squared))
    mul_from_real(x.second, q.second.abs_squared + q.first.abs_squared)
    x.second * Complex.from_real(q.second.abs_squared + q.first.abs_squared) =
        Complex.new(x.second.re * (q.second.abs_squared + q.first.abs_squared),
            x.second.im * (q.second.abs_squared + q.first.abs_squared))
    x.second * Complex.from_real(q.second.abs_squared) +
        x.second * Complex.from_real(q.first.abs_squared) =
        x.second * Complex.from_real(q.second.abs_squared + q.first.abs_squared)
    quat_norm(q) = norm2(q.first, q.second)
    norm2(q.first, q.second) = q.first.abs_squared + q.second.abs_squared
    q.second.abs_squared + q.first.abs_squared = q.first.abs_squared + q.second.abs_squared
    Complex.from_real(q.second.abs_squared + q.first.abs_squared) =
        Complex.from_real(quat_norm(q))
    x.second * Complex.from_real(q.second.abs_squared + q.first.abs_squared) =
        x.second * Complex.from_real(quat_norm(q))
    x.second * q.second.conj * q.second + x.second * q.first.conj * q.first =
        x.second * Complex.from_real(quat_norm(q))
    qmul_w(qmul_z(x.first, x.second, q.first, q.second),
        qmul_w(x.first, x.second, q.first, q.second),
        q.first.conj, -(q.second)) =
        x.second * Complex.from_real(quat_norm(q))
}

/// (x·q)·conj(q) scales x by the norm of q.
theorem quat_mul_right_conj(x: Quat, q: Quat) {
    quat_mul(quat_mul(x, q), quat_conj(q)) =
        Quat.new(x.first * Complex.from_real(quat_norm(q)),
            x.second * Complex.from_real(quat_norm(q)))
} by {
    quat_mul(quat_mul(x, q), quat_conj(q)) =
        Quat.new(qmul_z(quat_mul(x, q).first, quat_mul(x, q).second,
                quat_conj(q).first, quat_conj(q).second),
            qmul_w(quat_mul(x, q).first, quat_mul(x, q).second,
                quat_conj(q).first, quat_conj(q).second))
    quat_conj(q) = Quat.new(q.first.conj, -(q.second))
    quat_conj(q).first = q.first.conj
    quat_conj(q).second = -(q.second)
    quat_mul(x, q).first = qmul_z(x.first, x.second, q.first, q.second)
    quat_mul(x, q).second = qmul_w(x.first, x.second, q.first, q.second)
    quat_mul(quat_mul(x, q), quat_conj(q)) =
        Quat.new(qmul_z(qmul_z(x.first, x.second, q.first, q.second),
                qmul_w(x.first, x.second, q.first, q.second),
                q.first.conj, -(q.second)),
            qmul_w(qmul_z(x.first, x.second, q.first, q.second),
                qmul_w(x.first, x.second, q.first, q.second),
                q.first.conj, -(q.second)))
    quat_mul_right_conj_first(x, q)
    qmul_z(qmul_z(x.first, x.second, q.first, q.second),
        qmul_w(x.first, x.second, q.first, q.second),
        q.first.conj, -(q.second)) = x.first * Complex.from_real(quat_norm(q))
    quat_mul_right_conj_second(x, q)
    qmul_w(qmul_z(x.first, x.second, q.first, q.second),
        qmul_w(x.first, x.second, q.first, q.second),
        q.first.conj, -(q.second)) = x.second * Complex.from_real(quat_norm(q))
    quat_mul(quat_mul(x, q), quat_conj(q)) =
        Quat.new(x.first * Complex.from_real(quat_norm(q)),
            x.second * Complex.from_real(quat_norm(q)))
}

/// For a unit quaternion, (x·q)·conj(q) = x: right multiplication by conj(q)
/// undoes right multiplication by q.
theorem quat_unit_right_conj(x: Quat, q: Quat) {
    quat_norm(q) = Real.1 implies quat_mul(quat_mul(x, q), quat_conj(q)) = x
} by {
    if quat_norm(q) = Real.1 {
        quat_mul_right_conj(x, q)
        quat_mul(quat_mul(x, q), quat_conj(q)) =
            Quat.new(x.first * Complex.from_real(quat_norm(q)),
                x.second * Complex.from_real(quat_norm(q)))
        Complex.from_real(Real.1) = Complex.1
        quat_norm(q) = Real.1
        Complex.from_real(quat_norm(q)) = Complex.1
        Quat.new(x.first * Complex.from_real(quat_norm(q)),
            x.second * Complex.from_real(quat_norm(q))) =
            Quat.new(x.first * Complex.1, x.second * Complex.1)
        x.first * Complex.1 = x.first
        x.second * Complex.1 = x.second
        Quat.new(x.first * Complex.1, x.second * Complex.1) = Quat.new(x.first, x.second)
        Quat.new(x.first, x.second) = x
        quat_mul(quat_mul(x, q), quat_conj(q)) = x
    }
}

/// Right multiplication by a unit quaternion is injective.
theorem quat_mul_right_inj(x: Quat, y: Quat, q: Quat) {
    quat_norm(q) = Real.1 and quat_mul(x, q) = quat_mul(y, q) implies x = y
} by {
    if quat_norm(q) = Real.1 and quat_mul(x, q) = quat_mul(y, q) {
        quat_mul(quat_mul(x, q), quat_conj(q)) = quat_mul(quat_mul(y, q), quat_conj(q))
        quat_unit_right_conj(x, q)
        quat_mul(quat_mul(x, q), quat_conj(q)) = x
        quat_unit_right_conj(y, q)
        quat_mul(quat_mul(y, q), quat_conj(q)) = y
        x = y
    }
}

// ============================================================================
// The unit-distance count of a finite set of quaternions
// ============================================================================
//
// For a finite set s of quaternions, nu(s) is the number of unordered pairs
// at unit distance.  Right multiplication by a unit quaternion q is a
// bijection that preserves exactly the unit pairs (`quat_unit_dist_iff`), so
// nu is invariant: nu(s·q) = nu(s).  This is the counting engine of the
// 2026 disproof: it lets one replace a set by any of its unit-quaternion
// translates without changing the number of unit distances.

/// Right multiplication by q, as a function on quaternions.
define mul_right_q(q: Quat, x: Quat) -> Quat {
    quat_mul(x, q)
}

/// The translate of a set of quaternions by right multiplication by q.
define quat_set_mul(s: FiniteSet[Quat], q: Quat) -> FiniteSet[Quat] {
    fs_image(s, mul_right_q(q))
}

/// The pairwise translate: (x, y) ↦ (x·q, y·q).
define pair_mul_right_q(q: Quat, p: Pair[Quat, Quat]) -> Pair[Quat, Quat] {
    Pair.new(quat_mul(p.first, q), quat_mul(p.second, q))
}

/// True of an ordered pair of quaternions at unit distance.
define ordered_unit_pair(p: Pair[Quat, Quat]) -> Bool {
    quaternion_unit_distance(p.first, p.second)
}

/// The ordered pairs of points of s at unit distance.
define ordered_unit_pairs(s: FiniteSet[Quat]) -> FiniteSet[Pair[Quat, Quat]] {
    finite_set_filter(finite_set_product(s, s), ordered_unit_pair)
}

/// The number of unordered unit pairs of the finite set s.
define nu(s: FiniteSet[Quat]) -> Nat {
    fs_card(ordered_unit_pairs(s)).div(Nat.2)
}

/// Right multiplication by a unit quaternion is an injective function.
theorem mul_right_q_injective(q: Quat) {
    quat_norm(q) = Real.1 implies is_injective_fn(mul_right_q(q))
} by {
    if quat_norm(q) = Real.1 {
        forall(x: Quat, y: Quat) {
            if mul_right_q(q, x) = mul_right_q(q, y) {
                quat_mul(x, q) = quat_mul(y, q)
                quat_mul_right_inj(x, y, q)
                quat_norm(q) = Real.1 and quat_mul(x, q) = quat_mul(y, q)
                x = y
            }
            mul_right_q(q, x) = mul_right_q(q, y) implies x = y
        }
        is_injective_fn(mul_right_q(q)) = forall(x: Quat, y: Quat) {
            mul_right_q(q, x) = mul_right_q(q, y) implies x = y
        }
        is_injective_fn(mul_right_q(q))
    }
}

/// The pairwise translate by a unit quaternion is an injective function.
theorem pair_mul_right_q_injective(q: Quat) {
    quat_norm(q) = Real.1 implies is_injective_fn(pair_mul_right_q(q))
} by {
    if quat_norm(q) = Real.1 {
        forall(p: Pair[Quat, Quat], r: Pair[Quat, Quat]) {
            if pair_mul_right_q(q, p) = pair_mul_right_q(q, r) {
                pair_mul_right_q(q, p) = Pair.new(quat_mul(p.first, q), quat_mul(p.second, q))
                pair_mul_right_q(q, r) = Pair.new(quat_mul(r.first, q), quat_mul(r.second, q))
                Pair.new(quat_mul(p.first, q), quat_mul(p.second, q)) =
                    Pair.new(quat_mul(r.first, q), quat_mul(r.second, q))
                pair_new_first(quat_mul(p.first, q), quat_mul(p.second, q))
                Pair.new(quat_mul(p.first, q), quat_mul(p.second, q)).first = quat_mul(p.first, q)
                pair_new_first(quat_mul(r.first, q), quat_mul(r.second, q))
                Pair.new(quat_mul(r.first, q), quat_mul(r.second, q)).first = quat_mul(r.first, q)
                quat_mul(p.first, q) = quat_mul(r.first, q)
                quat_mul_right_inj(p.first, r.first, q)
                quat_norm(q) = Real.1 and quat_mul(p.first, q) = quat_mul(r.first, q)
                p.first = r.first
                pair_new_second(quat_mul(p.first, q), quat_mul(p.second, q))
                Pair.new(quat_mul(p.first, q), quat_mul(p.second, q)).second = quat_mul(p.second, q)
                pair_new_second(quat_mul(r.first, q), quat_mul(r.second, q))
                Pair.new(quat_mul(r.first, q), quat_mul(r.second, q)).second = quat_mul(r.second, q)
                Pair.new(quat_mul(p.first, q), quat_mul(p.second, q)) =
                    Pair.new(quat_mul(r.first, q), quat_mul(r.second, q))
                quat_mul(p.second, q) = quat_mul(r.second, q)
                quat_mul_right_inj(p.second, r.second, q)
                quat_norm(q) = Real.1 and quat_mul(p.second, q) = quat_mul(r.second, q)
                p.second = r.second
                pair_ext(p, r)
                p.first = r.first and p.second = r.second
                p = r
            }
            pair_mul_right_q(q, p) = pair_mul_right_q(q, r) implies p = r
        }
        is_injective_fn(pair_mul_right_q(q)) = forall(p: Pair[Quat, Quat], r: Pair[Quat, Quat]) {
            pair_mul_right_q(q, p) = pair_mul_right_q(q, r) implies p = r
        }
        is_injective_fn(pair_mul_right_q(q))
    }
}

/// The quaternion unit-distance predicate is decided by the norm-one
/// difference: dist(x, y)² = 1 exactly when N(x - y) = 1.
theorem quat_unit_dist_norm(x: Quat, y: Quat) {
    quaternion_unit_distance(x, y) = (quat_norm(quat_sub(x, y)) = Real.1)
} by {
    quaternion_unit_distance(x, y) = (quat_dist_sq(x, y) = Real.1)
    quat_dist_sq(x, y) = quat_norm(quat_sub(x, y))
    quaternion_unit_distance(x, y) = (quat_norm(quat_sub(x, y)) = Real.1)
}

/// A unit pair of the translated set is the translate of a unit pair of s.
theorem oup_unit_translate_forward(s: FiniteSet[Quat], q: Quat, p: Pair[Quat, Quat]) {
    quat_norm(q) = Real.1 and ordered_unit_pairs(quat_set_mul(s, q)).contains(p)
    implies fs_image(ordered_unit_pairs(s), pair_mul_right_q(q)).contains(p)
} by {
    if quat_norm(q) = Real.1 and ordered_unit_pairs(quat_set_mul(s, q)).contains(p) {
        finite_set_filter_contains_eq(finite_set_product(quat_set_mul(s, q), quat_set_mul(s, q)),
            ordered_unit_pair, p)
        finite_set_filter(finite_set_product(quat_set_mul(s, q), quat_set_mul(s, q)),
            ordered_unit_pair).contains(p) =
            (finite_set_product(quat_set_mul(s, q), quat_set_mul(s, q)).contains(p) and
                ordered_unit_pair(p))
        ordered_unit_pairs(quat_set_mul(s, q)) =
            finite_set_filter(finite_set_product(quat_set_mul(s, q), quat_set_mul(s, q)),
                ordered_unit_pair)
        ordered_unit_pairs(quat_set_mul(s, q)).contains(p) =
            finite_set_filter(finite_set_product(quat_set_mul(s, q), quat_set_mul(s, q)),
                ordered_unit_pair).contains(p)
        finite_set_filter(finite_set_product(quat_set_mul(s, q), quat_set_mul(s, q)),
            ordered_unit_pair).contains(p)
        finite_set_product(quat_set_mul(s, q), quat_set_mul(s, q)).contains(p) and
            ordered_unit_pair(p)
        finite_set_product_contains_eq(quat_set_mul(s, q), quat_set_mul(s, q), p)
        finite_set_product(quat_set_mul(s, q), quat_set_mul(s, q)).contains(p) =
            (quat_set_mul(s, q).contains(p.first) and quat_set_mul(s, q).contains(p.second))
        quat_set_mul(s, q).contains(p.first) and quat_set_mul(s, q).contains(p.second)
        quat_set_mul(s, q).contains(p.first)
        finite_set_image_contains_eq(s, mul_right_q(q), p.first)
        fs_image(s, mul_right_q(q)).contains(p.first) = exists(x: Quat) {
            s.contains(x) and p.first = mul_right_q(q, x)
        }
        exists(x: Quat) {
            s.contains(x) and p.first = mul_right_q(q, x)
        }
        let (x: Quat) satisfy {
            s.contains(x) and p.first = mul_right_q(q, x)
        }
        s.contains(x)
        p.first = mul_right_q(q, x)
        quat_set_mul(s, q).contains(p.second)
        finite_set_image_contains_eq(s, mul_right_q(q), p.second)
        fs_image(s, mul_right_q(q)).contains(p.second) = exists(y: Quat) {
            s.contains(y) and p.second = mul_right_q(q, y)
        }
        exists(y: Quat) {
            s.contains(y) and p.second = mul_right_q(q, y)
        }
        let (y: Quat) satisfy {
            s.contains(y) and p.second = mul_right_q(q, y)
        }
        s.contains(y)
        p.second = mul_right_q(q, y)
        mul_right_q(q, x) = quat_mul(x, q)
        p.first = quat_mul(x, q)
        mul_right_q(q, y) = quat_mul(y, q)
        p.second = quat_mul(y, q)
        ordered_unit_pair(p)
        quaternion_unit_distance(p.first, p.second)
        quat_dist_sq(p.first, p.second) = Real.1
        quat_unit_dist_iff(x, y, q)
        quaternion_unit_distance(x, y) = quaternion_unit_distance(quat_mul(x, q), quat_mul(y, q))
        quat_dist_sq(quat_mul(x, q), quat_mul(y, q)) = Real.1
        p.first = quat_mul(x, q)
        p.second = quat_mul(y, q)
        quat_dist_sq(p.first, p.second) = quat_dist_sq(quat_mul(x, q), quat_mul(y, q))
        quaternion_unit_distance(quat_mul(x, q), quat_mul(y, q))
        quaternion_unit_distance(x, y)
        ordered_unit_pair(Pair.new(x, y))
        finite_set_product_contains_pair(s, s, x, y)
        s.contains(x) and s.contains(y)
        finite_set_product(s, s).contains(Pair.new(x, y))
        finite_set_filter_contains_eq(finite_set_product(s, s), ordered_unit_pair, Pair.new(x, y))
        finite_set_filter(finite_set_product(s, s), ordered_unit_pair).contains(Pair.new(x, y)) =
            (finite_set_product(s, s).contains(Pair.new(x, y)) and ordered_unit_pair(Pair.new(x, y)))
        ordered_unit_pairs(s).contains(Pair.new(x, y))
        pair_mul_right_q(q, Pair.new(x, y)) = Pair.new(quat_mul(x, q), quat_mul(y, q))
        Pair.new(p.first, p.second) = p
        p = Pair.new(quat_mul(x, q), quat_mul(y, q))
        p = pair_mul_right_q(q, Pair.new(x, y))
        finite_set_image_contains_eq(ordered_unit_pairs(s), pair_mul_right_q(q), p)
        fs_image(ordered_unit_pairs(s), pair_mul_right_q(q)).contains(p) = exists(w: Pair[Quat, Quat]) {
            ordered_unit_pairs(s).contains(w) and p = pair_mul_right_q(q, w)
        }
        exists(w: Pair[Quat, Quat]) {
            ordered_unit_pairs(s).contains(w) and p = pair_mul_right_q(q, w)
        }
        fs_image(ordered_unit_pairs(s), pair_mul_right_q(q)).contains(p)
    }
}

/// The translate of a unit pair of s is a unit pair of the translated set.
theorem oup_unit_translate_reverse(s: FiniteSet[Quat], q: Quat, p: Pair[Quat, Quat]) {
    quat_norm(q) = Real.1 and fs_image(ordered_unit_pairs(s), pair_mul_right_q(q)).contains(p)
    implies ordered_unit_pairs(quat_set_mul(s, q)).contains(p)
} by {
    if quat_norm(q) = Real.1 and fs_image(ordered_unit_pairs(s), pair_mul_right_q(q)).contains(p) {
        finite_set_image_contains_eq(ordered_unit_pairs(s), pair_mul_right_q(q), p)
        fs_image(ordered_unit_pairs(s), pair_mul_right_q(q)).contains(p) = exists(w: Pair[Quat, Quat]) {
            ordered_unit_pairs(s).contains(w) and p = pair_mul_right_q(q, w)
        }
        exists(w: Pair[Quat, Quat]) {
            ordered_unit_pairs(s).contains(w) and p = pair_mul_right_q(q, w)
        }
        let (w: Pair[Quat, Quat]) satisfy {
            ordered_unit_pairs(s).contains(w) and p = pair_mul_right_q(q, w)
        }
        ordered_unit_pairs(s).contains(w)
        p = pair_mul_right_q(q, w)
        finite_set_filter_contains_eq(finite_set_product(s, s), ordered_unit_pair, w)
        finite_set_filter(finite_set_product(s, s), ordered_unit_pair).contains(w) =
            (finite_set_product(s, s).contains(w) and ordered_unit_pair(w))
        ordered_unit_pairs(s) = finite_set_filter(finite_set_product(s, s), ordered_unit_pair)
        ordered_unit_pairs(s).contains(w) =
            finite_set_filter(finite_set_product(s, s), ordered_unit_pair).contains(w)
        finite_set_filter(finite_set_product(s, s), ordered_unit_pair).contains(w)
        finite_set_product(s, s).contains(w) and ordered_unit_pair(w)
        finite_set_product(s, s).contains(w)
        ordered_unit_pair(w)
        finite_set_product_contains_eq(s, s, w)
        finite_set_product(s, s).contains(w) = (s.contains(w.first) and s.contains(w.second))
        s.contains(w.first) and s.contains(w.second)
        s.contains(w.first)
        s.contains(w.second)
        pair_mul_right_q(q, w) = Pair.new(quat_mul(w.first, q), quat_mul(w.second, q))
        p = Pair.new(quat_mul(w.first, q), quat_mul(w.second, q))
        Pair.new(p.first, p.second) = p
        pair_new_first(quat_mul(w.first, q), quat_mul(w.second, q))
        Pair.new(quat_mul(w.first, q), quat_mul(w.second, q)).first = quat_mul(w.first, q)
        p.first = quat_mul(w.first, q)
        pair_new_second(quat_mul(w.first, q), quat_mul(w.second, q))
        Pair.new(quat_mul(w.first, q), quat_mul(w.second, q)).second = quat_mul(w.second, q)
        p.second = quat_mul(w.second, q)
        quaternion_unit_distance(w.first, w.second)
        quat_unit_dist_iff(w.first, w.second, q)
        quaternion_unit_distance(w.first, w.second) =
            quaternion_unit_distance(quat_mul(w.first, q), quat_mul(w.second, q))
        quaternion_unit_distance(quat_mul(w.first, q), quat_mul(w.second, q))
        p.first = quat_mul(w.first, q)
        p.second = quat_mul(w.second, q)
        quaternion_unit_distance(p.first, p.second)
        ordered_unit_pair(p)
        finite_set_image_contains_eq(s, mul_right_q(q), p.first)
        fs_image(s, mul_right_q(q)).contains(p.first) = exists(x: Quat) {
            s.contains(x) and p.first = mul_right_q(q, x)
        }
        mul_right_q(q, w.first) = quat_mul(w.first, q)
        s.contains(w.first) and p.first = mul_right_q(q, w.first)
        exists(x: Quat) {
            s.contains(x) and p.first = mul_right_q(q, x)
        }
        quat_set_mul(s, q).contains(p.first)
        finite_set_image_contains_eq(s, mul_right_q(q), p.second)
        fs_image(s, mul_right_q(q)).contains(p.second) = exists(y: Quat) {
            s.contains(y) and p.second = mul_right_q(q, y)
        }
        mul_right_q(q, w.second) = quat_mul(w.second, q)
        s.contains(w.second) and p.second = mul_right_q(q, w.second)
        exists(y: Quat) {
            s.contains(y) and p.second = mul_right_q(q, y)
        }
        quat_set_mul(s, q).contains(p.second)
        finite_set_product_contains_eq(quat_set_mul(s, q), quat_set_mul(s, q), p)
        finite_set_product(quat_set_mul(s, q), quat_set_mul(s, q)).contains(p) =
            (quat_set_mul(s, q).contains(p.first) and quat_set_mul(s, q).contains(p.second))
        quat_set_mul(s, q).contains(p.first) and quat_set_mul(s, q).contains(p.second)
        finite_set_product(quat_set_mul(s, q), quat_set_mul(s, q)).contains(p)
        finite_set_filter_contains_eq(finite_set_product(quat_set_mul(s, q), quat_set_mul(s, q)),
            ordered_unit_pair, p)
        finite_set_filter(finite_set_product(quat_set_mul(s, q), quat_set_mul(s, q)),
            ordered_unit_pair).contains(p) =
            (finite_set_product(quat_set_mul(s, q), quat_set_mul(s, q)).contains(p) and
                ordered_unit_pair(p))
        ordered_unit_pairs(quat_set_mul(s, q)) =
            finite_set_filter(finite_set_product(quat_set_mul(s, q), quat_set_mul(s, q)),
                ordered_unit_pair)
        ordered_unit_pairs(quat_set_mul(s, q)).contains(p) =
            finite_set_filter(finite_set_product(quat_set_mul(s, q), quat_set_mul(s, q)),
                ordered_unit_pair).contains(p)
        finite_set_product(quat_set_mul(s, q), quat_set_mul(s, q)).contains(p) and
            ordered_unit_pair(p)
        finite_set_filter(finite_set_product(quat_set_mul(s, q), quat_set_mul(s, q)),
            ordered_unit_pair).contains(p)
        ordered_unit_pairs(quat_set_mul(s, q)).contains(p)
    }
}

/// The ordered unit pairs of the translated set are the translates of the
/// ordered unit pairs of s.
theorem ordered_unit_pairs_unit_translate(s: FiniteSet[Quat], q: Quat) {
    quat_norm(q) = Real.1 implies
        ordered_unit_pairs(quat_set_mul(s, q)) = fs_image(ordered_unit_pairs(s), pair_mul_right_q(q))
} by {
    if quat_norm(q) = Real.1 {
        forall(p: Pair[Quat, Quat]) {
            oup_unit_translate_forward(s, q, p)
            ordered_unit_pairs(quat_set_mul(s, q)).contains(p) implies fs_image(ordered_unit_pairs(s), pair_mul_right_q(q)).contains(p)
            oup_unit_translate_reverse(s, q, p)
            fs_image(ordered_unit_pairs(s), pair_mul_right_q(q)).contains(p) implies ordered_unit_pairs(quat_set_mul(s, q)).contains(p)
            ordered_unit_pairs(quat_set_mul(s, q)).contains(p) =
                fs_image(ordered_unit_pairs(s), pair_mul_right_q(q)).contains(p)
        }
        forall(p: Pair[Quat, Quat]) {
            ordered_unit_pairs(quat_set_mul(s, q)).underlying_set.contains(p) =
                fs_image(ordered_unit_pairs(s), pair_mul_right_q(q)).underlying_set.contains(p)
        }
        finite_set_ext_contains(ordered_unit_pairs(quat_set_mul(s, q)),
            fs_image(ordered_unit_pairs(s), pair_mul_right_q(q)))
        ordered_unit_pairs(quat_set_mul(s, q)) =
            fs_image(ordered_unit_pairs(s), pair_mul_right_q(q))
    }
}

/// The translated set has as many ordered unit pairs as the original.
theorem oup_card_unit_translate(s: FiniteSet[Quat], q: Quat) {
    quat_norm(q) = Real.1 implies
        fs_card(ordered_unit_pairs(quat_set_mul(s, q))) = fs_card(ordered_unit_pairs(s))
} by {
    if quat_norm(q) = Real.1 {
        ordered_unit_pairs_unit_translate(s, q)
        ordered_unit_pairs(quat_set_mul(s, q)) =
            fs_image(ordered_unit_pairs(s), pair_mul_right_q(q))
        pair_mul_right_q_injective(q)
        quat_norm(q) = Real.1
        is_injective_fn(pair_mul_right_q(q))
        fs_card_cardinality_is(ordered_unit_pairs(s))
        ordered_unit_pairs(s).cardinality_is(fs_card(ordered_unit_pairs(s)))
        finite_set_image_cardinality_is_of_injective(ordered_unit_pairs(s),
            pair_mul_right_q(q), fs_card(ordered_unit_pairs(s)))
        is_injective_fn(pair_mul_right_q(q)) and
            ordered_unit_pairs(s).cardinality_is(fs_card(ordered_unit_pairs(s)))
        fs_image(ordered_unit_pairs(s), pair_mul_right_q(q)).cardinality_is(fs_card(ordered_unit_pairs(s)))
        fs_card_eq_of_cardinality_is(fs_image(ordered_unit_pairs(s), pair_mul_right_q(q)),
            fs_card(ordered_unit_pairs(s)))
        fs_card(fs_image(ordered_unit_pairs(s), pair_mul_right_q(q))) = fs_card(ordered_unit_pairs(s))
        fs_card(ordered_unit_pairs(quat_set_mul(s, q))) = fs_card(ordered_unit_pairs(s))
    }
}

/// The number of unit distances is invariant under multiplication by a unit
/// quaternion: nu(s·q) = nu(s) whenever N(q) = 1.
theorem nu_unit_invariant(s: FiniteSet[Quat], q: Quat) {
    quat_norm(q) = Real.1 implies nu(quat_set_mul(s, q)) = nu(s)
} by {
    if quat_norm(q) = Real.1 {
        oup_card_unit_translate(s, q)
        fs_card(ordered_unit_pairs(quat_set_mul(s, q))) = fs_card(ordered_unit_pairs(s))
        nu(quat_set_mul(s, q)) = fs_card(ordered_unit_pairs(quat_set_mul(s, q))).div(Nat.2)
        nu(s) = fs_card(ordered_unit_pairs(s)).div(Nat.2)
        nu(quat_set_mul(s, q)) = nu(s)
    }
}

/// The empty set has no unit distances.
theorem nu_empty {
    nu(FiniteSet.empty[Quat]) = Nat.0
} by {
    nu(FiniteSet.empty[Quat]) = fs_card(ordered_unit_pairs(FiniteSet.empty[Quat])).div(Nat.2)
    ordered_unit_pairs(FiniteSet.empty[Quat]) =
        finite_set_filter(finite_set_product(FiniteSet.empty[Quat], FiniteSet.empty[Quat]), ordered_unit_pair)
    finite_set_product(FiniteSet.empty[Quat], FiniteSet.empty[Quat]) = FiniteSet.empty[Pair[Quat, Quat]]
    ordered_unit_pairs(FiniteSet.empty[Quat]) =
        finite_set_filter(FiniteSet.empty[Pair[Quat, Quat]], ordered_unit_pair)
    finite_set_filter(FiniteSet.empty[Pair[Quat, Quat]], ordered_unit_pair) =
        FiniteSet.empty[Pair[Quat, Quat]]
    fs_card(FiniteSet.empty[Pair[Quat, Quat]]) = Nat.0
    fs_card(ordered_unit_pairs(FiniteSet.empty[Quat])) = Nat.0
    nu(FiniteSet.empty[Quat]) = Nat.0.div(Nat.2)
    Nat.0.div(Nat.2) = Nat.0
    nu(FiniteSet.empty[Quat]) = Nat.0
}

// ============================================================================
// Vector addition and the counting structure of nu
// ============================================================================
//
// The counting mechanism of the 2026 disproof turns many unit-norm vectors
// into many unit distances: if every point of a window has all of its
// translates by a set `u` of unit vectors inside the point set, then the
// set carries `|u|` unit distances per window point.  For that one needs
// vector addition on quaternions (`quat_add`) and the fact that a unit
// translate keeps the squared distance equal to the norm of the vector
// (`quat_dist_sq_add_right`).  The section then records the structural
// facts of the count `nu`: unit distance is symmetric
// (`quat_unit_dist_comm`), no point is at unit distance from itself, so the
// ordered unit pairs come in swapped pairs (`oup_swap_member`,
// `oup_no_self_member`), and the ordered unit pairs are a subset of all
// ordered pairs (`oup_card_at_most_sq`).

/// The sum of two quaternions (vector addition in R⁴).
define quat_add(x: Quat, y: Quat) -> Quat {
    Quat.new(x.first + y.first, x.second + y.second)
}

/// Adding the same quaternion to both sides cancels on the left.
theorem quat_add_left_cancel(a: Quat, w: Quat, v: Quat) {
    quat_add(a, w) = quat_add(a, v) implies w = v
} by {
    if quat_add(a, w) = quat_add(a, v) {
        quat_add(a, w) = Quat.new(a.first + w.first, a.second + w.second)
        quat_add(a, v) = Quat.new(a.first + v.first, a.second + v.second)
        Quat.new(a.first + w.first, a.second + w.second) =
            Quat.new(a.first + v.first, a.second + v.second)
        pair_new_first(a.first + w.first, a.second + w.second)
        Quat.new(a.first + w.first, a.second + w.second).first = a.first + w.first
        pair_new_first(a.first + v.first, a.second + v.second)
        Quat.new(a.first + v.first, a.second + v.second).first = a.first + v.first
        a.first + w.first = a.first + v.first
        left_cancel(a.first, w.first, v.first)
        w.first = v.first
        pair_new_second(a.first + w.first, a.second + w.second)
        Quat.new(a.first + w.first, a.second + w.second).second = a.second + w.second
        pair_new_second(a.first + v.first, a.second + v.second)
        Quat.new(a.first + v.first, a.second + v.second).second = a.second + v.second
        Quat.new(a.first + w.first, a.second + w.second) =
            Quat.new(a.first + v.first, a.second + v.second)
        a.second + w.second = a.second + v.second
        left_cancel(a.second, w.second, v.second)
        w.second = v.second
        Quat.new(w.first, w.second) = w
        Quat.new(v.first, v.second) = v
        w = v
    }
}

/// The negative of a quaternion.
define quat_neg(x: Quat) -> Quat {
    Quat.new(-x.first, -x.second)
}

/// Subtracting a sum from its first summand leaves the negative of the
/// second: x - (x + y) = -y.
theorem complex_sub_add_right(x: Complex, y: Complex) {
    x - (x + y) = -y
} by {
    x - (x + y) = x + -(x + y)
    inverse_add(x, y)
    -(x + y) = -y + -x
    x + -(x + y) = x + (-y + -x)
    add_comm(-y, -x)
    -y + -x = -x + -y
    x + (-y + -x) = x + (-x + -y)
    add_comm(x, -x)
    x + -x = -x + x
    inverse_left(x)
    -x + x = Complex.0
    x + -x = Complex.0
    x + (-x + -y) = (x + -x) + -y
    (x + -x) + -y = Complex.0 + -y
    Complex.0 + -y = -y
    x - (x + y) = -y
}

/// Subtracting a sum from its first summand leaves the negative of the
/// second: a - (a + w) = -w.
theorem quat_sub_add_right(a: Quat, w: Quat) {
    quat_sub(a, quat_add(a, w)) = quat_neg(w)
} by {
    quat_sub(a, quat_add(a, w)) =
        Quat.new(a.first - quat_add(a, w).first, a.second - quat_add(a, w).second)
    quat_add(a, w) = Quat.new(a.first + w.first, a.second + w.second)
    quat_add(a, w).first = a.first + w.first
    quat_add(a, w).second = a.second + w.second
    quat_sub(a, quat_add(a, w)) =
        Quat.new(a.first - (a.first + w.first), a.second - (a.second + w.second))
    a.first - (a.first + w.first) = -w.first
    a.second - (a.second + w.second) = -w.second
    Quat.new(a.first - (a.first + w.first), a.second - (a.second + w.second)) =
        Quat.new(-w.first, -w.second)
    quat_neg(w) = Quat.new(-w.first, -w.second)
    quat_sub(a, quat_add(a, w)) = quat_neg(w)
}

/// The norm of a quaternion is unchanged by negation.
theorem quat_norm_neg(x: Quat) {
    quat_norm(quat_neg(x)) = quat_norm(x)
} by {
    quat_norm(quat_neg(x)) = norm2(quat_neg(x).first, quat_neg(x).second)
    quat_neg(x) = Quat.new(-x.first, -x.second)
    quat_neg(x).first = -x.first
    quat_neg(x).second = -x.second
    quat_norm(quat_neg(x)) = norm2(-x.first, -x.second)
    norm2(-x.first, -x.second) = (-x.first).abs_squared + (-x.second).abs_squared
    abs_squared_neg(x.first)
    (-x.first).abs_squared = x.first.abs_squared
    abs_squared_neg(x.second)
    (-x.second).abs_squared = x.second.abs_squared
    (-x.first).abs_squared + (-x.second).abs_squared =
        x.first.abs_squared + x.second.abs_squared
    quat_norm(x) = norm2(x.first, x.second)
    norm2(x.first, x.second) = x.first.abs_squared + x.second.abs_squared
    quat_norm(quat_neg(x)) = quat_norm(x)
}

/// Subtracting the added element recovers the first summand:
/// (x + y) - y = x.
theorem complex_add_sub_cancel(x: Complex, y: Complex) {
    (x + y) - y = x
} by {
    (x + y) - y = (x + y) + -y
    add_comm(y, -y)
    y + -y = -y + y
    inverse_left(y)
    -y + y = Complex.0
    y + -y = Complex.0
    (x + y) + -y = x + (y + -y)
    x + (y + -y) = x + Complex.0
    x + Complex.0 = x
    (x + y) - y = x
}

/// Subtracting the first summand of a sum recovers the second: (a + w) - a = w.
theorem quat_sub_add_left(a: Quat, w: Quat) {
    quat_sub(quat_add(a, w), a) = w
} by {
    quat_sub(quat_add(a, w), a) =
        Quat.new(quat_add(a, w).first - a.first, quat_add(a, w).second - a.second)
    quat_add(a, w) = Quat.new(a.first + w.first, a.second + w.second)
    quat_add(a, w).first = a.first + w.first
    quat_add(a, w).second = a.second + w.second
    quat_sub(quat_add(a, w), a) =
        Quat.new((a.first + w.first) - a.first, (a.second + w.second) - a.second)
    add_comm(w.first, a.first)
    w.first + a.first = a.first + w.first
    complex_add_sub_cancel(w.first, a.first)
    (w.first + a.first) - a.first = w.first
    (a.first + w.first) - a.first = w.first
    add_comm(w.second, a.second)
    w.second + a.second = a.second + w.second
    complex_add_sub_cancel(w.second, a.second)
    (w.second + a.second) - a.second = w.second
    (a.second + w.second) - a.second = w.second
    Quat.new((a.first + w.first) - a.first, (a.second + w.second) - a.second) =
        Quat.new(w.first, w.second)
    Quat.new(w.first, w.second) = w
    quat_sub(quat_add(a, w), a) = w
}

/// The squared distance from a point to its translate by w is the norm of w.
theorem quat_dist_sq_add_right(a: Quat, w: Quat) {
    quat_dist_sq(a, quat_add(a, w)) = quat_norm(w)
} by {
    quat_dist_sq(a, quat_add(a, w)) = quat_norm(quat_sub(a, quat_add(a, w)))
    quat_sub_add_right(a, w)
    quat_sub(a, quat_add(a, w)) = quat_neg(w)
    quat_dist_sq(a, quat_add(a, w)) = quat_norm(quat_neg(w))
    quat_norm_neg(w)
    quat_norm(quat_neg(w)) = quat_norm(w)
    quat_dist_sq(a, quat_add(a, w)) = quat_norm(w)
}

/// Zero is not one in the reals.
theorem real_zero_ne_one {
    Real.0 != Real.1
} by {
    if Real.0 = Real.1 {
        from_nat_real_pos_of_ne_zero(Nat.1)
        Nat.1 != Nat.0
        from_nat[Real](Nat.1) > Real.0
        from_nat_one[Real]
        from_nat[Real](Nat.1) = Real.1
        Real.1 > Real.0
        Real.0 > Real.0
        lt_not_ref(Real.0)
        false
    }
    Real.0 != Real.1
}

/// A quaternion is at squared distance zero from itself.
theorem quat_dist_sq_self(x: Quat) {
    quat_dist_sq(x, x) = Real.0
} by {
    quat_dist_sq(x, x) = quat_norm(quat_sub(x, x))
    quat_sub(x, x) = Quat.new(x.first - x.first, x.second - x.second)
    quat_sub(x, x).first = x.first - x.first
    quat_sub(x, x).second = x.second - x.second
    quat_norm(quat_sub(x, x)) = norm2(quat_sub(x, x).first, quat_sub(x, x).second)
    x.first - x.first = x.first + -(x.first)
    add_comm(x.first, -(x.first))
    x.first + -(x.first) = -(x.first) + x.first
    inverse_left(x.first)
    -(x.first) + x.first = Complex.0
    x.first - x.first = Complex.0
    x.second - x.second = x.second + -(x.second)
    add_comm(x.second, -(x.second))
    x.second + -(x.second) = -(x.second) + x.second
    inverse_left(x.second)
    -(x.second) + x.second = Complex.0
    x.second - x.second = Complex.0
    quat_sub(x, x) = Quat.new(Complex.0, Complex.0)
    quat_norm(quat_sub(x, x)) = norm2(Complex.0, Complex.0)
    norm2(Complex.0, Complex.0) = Complex.0.abs_squared + Complex.0.abs_squared
    abs_squared_eq(Complex.0)
    Complex.0.abs_squared = Complex.0.re * Complex.0.re + Complex.0.im * Complex.0.im
    Complex.0.re * Complex.0.re + Complex.0.im * Complex.0.im = Real.0 * Real.0 + Real.0 * Real.0
    Real.0 * Real.0 + Real.0 * Real.0 = Real.0
    Complex.0.abs_squared = Real.0
    Complex.0.abs_squared = Real.0
    norm2(Complex.0, Complex.0) = Real.0 + Real.0
    Real.0 + Real.0 = Real.0
    quat_dist_sq(x, x) = Real.0
}

/// Negating a difference of complex numbers swaps its arguments.
theorem complex_neg_sub(a: Complex, b: Complex) {
    b - a = -(a - b)
} by {
    b - a = b + -(a)
    a - b = a + -(b)
    inverse_add(a, -(b))
    -(a + -(b)) = -(-(b)) + -(a)
    inverse_inverse(b)
    -(-(b)) = b
    -(a + -(b)) = b + -(a)
    -(a - b) = b + -(a)
    b - a = -(a - b)
}

/// The squared distance of two quaternions is symmetric.
theorem quat_dist_sq_comm(x: Quat, y: Quat) {
    quat_dist_sq(x, y) = quat_dist_sq(y, x)
} by {
    quat_dist_sq(x, y) = norm2(x.first - y.first, x.second - y.second)
    quat_dist_sq(y, x) = norm2(y.first - x.first, y.second - x.second)
    norm2(x.first - y.first, x.second - y.second) =
        (x.first - y.first).abs_squared + (x.second - y.second).abs_squared
    norm2(y.first - x.first, y.second - x.second) =
        (y.first - x.first).abs_squared + (y.second - x.second).abs_squared
    complex_neg_sub(x.first, y.first)
    y.first - x.first = -(x.first - y.first)
    abs_squared_neg(x.first - y.first)
    (-(x.first - y.first)).abs_squared = (x.first - y.first).abs_squared
    (y.first - x.first).abs_squared = (x.first - y.first).abs_squared
    complex_neg_sub(x.second, y.second)
    y.second - x.second = -(x.second - y.second)
    abs_squared_neg(x.second - y.second)
    (-(x.second - y.second)).abs_squared = (x.second - y.second).abs_squared
    (y.second - x.second).abs_squared = (x.second - y.second).abs_squared
    (x.first - y.first).abs_squared + (x.second - y.second).abs_squared =
        (y.first - x.first).abs_squared + (y.second - x.second).abs_squared
    quat_dist_sq(x, y) = quat_dist_sq(y, x)
}

/// Unit distance is symmetric: (x, y) is a unit pair exactly when (y, x) is.
theorem quat_unit_dist_comm(x: Quat, y: Quat) {
    quaternion_unit_distance(x, y) = quaternion_unit_distance(y, x)
} by {
    quaternion_unit_distance(x, y) = (quat_dist_sq(x, y) = Real.1)
    quaternion_unit_distance(y, x) = (quat_dist_sq(y, x) = Real.1)
    quat_dist_sq_comm(x, y)
    quat_dist_sq(x, y) = quat_dist_sq(y, x)
    quaternion_unit_distance(x, y) = quaternion_unit_distance(y, x)
}

/// Swapping the two points of a member of the ordered unit pairs keeps it a
/// member: the ordered unit pairs are closed under reversal.
theorem oup_swap_member(s: FiniteSet[Quat], x: Quat, y: Quat) {
    ordered_unit_pairs(s).contains(Pair.new(x, y)) =
        ordered_unit_pairs(s).contains(Pair.new(y, x))
} by {
    finite_set_filter_contains_eq(finite_set_product(s, s), ordered_unit_pair, Pair.new(x, y))
    finite_set_filter(finite_set_product(s, s), ordered_unit_pair).contains(Pair.new(x, y)) =
        (finite_set_product(s, s).contains(Pair.new(x, y)) and ordered_unit_pair(Pair.new(x, y)))
    ordered_unit_pairs(s) = finite_set_filter(finite_set_product(s, s), ordered_unit_pair)
    finite_set_filter_contains_eq(finite_set_product(s, s), ordered_unit_pair, Pair.new(y, x))
    finite_set_filter(finite_set_product(s, s), ordered_unit_pair).contains(Pair.new(y, x)) =
        (finite_set_product(s, s).contains(Pair.new(y, x)) and ordered_unit_pair(Pair.new(y, x)))
    finite_set_product_contains_eq(s, s, Pair.new(x, y))
    finite_set_product(s, s).contains(Pair.new(x, y)) =
        (s.contains(x) and s.contains(y))
    finite_set_product_contains_eq(s, s, Pair.new(y, x))
    finite_set_product(s, s).contains(Pair.new(y, x)) =
        (s.contains(y) and s.contains(x))
    ordered_unit_pair(Pair.new(x, y)) = quaternion_unit_distance(x, y)
    ordered_unit_pair(Pair.new(y, x)) = quaternion_unit_distance(y, x)
    quat_unit_dist_comm(x, y)
    quaternion_unit_distance(x, y) = quaternion_unit_distance(y, x)
    ordered_unit_pair(Pair.new(x, y)) = ordered_unit_pair(Pair.new(y, x))
    (s.contains(x) and s.contains(y)) = (s.contains(y) and s.contains(x))
    (finite_set_product(s, s).contains(Pair.new(x, y)) and ordered_unit_pair(Pair.new(x, y))) =
        (finite_set_product(s, s).contains(Pair.new(y, x)) and ordered_unit_pair(Pair.new(y, x)))
    ordered_unit_pairs(s).contains(Pair.new(x, y)) =
        ordered_unit_pairs(s).contains(Pair.new(y, x))
}

/// No point is at unit distance from itself, so no diagonal pair is a member
/// of the ordered unit pairs.
theorem oup_no_self_member(s: FiniteSet[Quat], x: Quat) {
    not ordered_unit_pairs(s).contains(Pair.new(x, x))
} by {
    if ordered_unit_pairs(s).contains(Pair.new(x, x)) {
        finite_set_filter_contains_eq(finite_set_product(s, s), ordered_unit_pair, Pair.new(x, x))
        finite_set_filter(finite_set_product(s, s), ordered_unit_pair).contains(Pair.new(x, x)) =
            (finite_set_product(s, s).contains(Pair.new(x, x)) and ordered_unit_pair(Pair.new(x, x)))
        ordered_unit_pairs(s) = finite_set_filter(finite_set_product(s, s), ordered_unit_pair)
        ordered_unit_pairs(s).contains(Pair.new(x, x)) =
            finite_set_filter(finite_set_product(s, s), ordered_unit_pair).contains(Pair.new(x, x))
        finite_set_filter(finite_set_product(s, s), ordered_unit_pair).contains(Pair.new(x, x))
        ordered_unit_pair(Pair.new(x, x))
        ordered_unit_pair(Pair.new(x, x)) = quaternion_unit_distance(x, x)
        quaternion_unit_distance(x, x)
        quaternion_unit_distance(x, x) = (quat_dist_sq(x, x) = Real.1)
        quat_dist_sq(x, x) = Real.1
        quat_dist_sq_self(x)
        quat_dist_sq(x, x) = Real.0
        Real.0 = Real.1
        real_zero_ne_one
        Real.0 != Real.1
        false
    }
    not ordered_unit_pairs(s).contains(Pair.new(x, x))
}

/// The ordered unit pairs of a finite set are a subset of all ordered pairs,
/// so their count is at most the square of the set's cardinality.
theorem oup_card_at_most_sq(s: FiniteSet[Quat]) {
    fs_card(ordered_unit_pairs(s)) <= fs_card(s) * fs_card(s)
} by {
    ordered_unit_pairs(s) = finite_set_filter(finite_set_product(s, s), ordered_unit_pair)
    finite_set_filter_subset(finite_set_product(s, s), ordered_unit_pair)
    finite_set_filter(finite_set_product(s, s), ordered_unit_pair).subset_eq(finite_set_product(s, s))
    ordered_unit_pairs(s).subset_eq(finite_set_product(s, s))
    fs_card_mono(ordered_unit_pairs(s), finite_set_product(s, s))
    fs_card(ordered_unit_pairs(s)) <= fs_card(finite_set_product(s, s))
    fs_card_product(s, s)
    fs_card(finite_set_product(s, s)) = fs_card(s) * fs_card(s)
    lte_trans_eq(fs_card(ordered_unit_pairs(s)), fs_card(finite_set_product(s, s)),
        fs_card(s) * fs_card(s))
    fs_card(ordered_unit_pairs(s)) <= fs_card(s) * fs_card(s)
}
