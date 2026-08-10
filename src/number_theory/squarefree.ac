// Squarefree natural numbers and their connections to the Mobius function.
//
// The predicate `is_squarefree` (no square other than one divides the number)
// lives in `data/nat/nat_squarefree.ac`, together with its divisor-theoretic
// consequences; `data/nat/nat_squarefree_prime.ac` gives the equivalent prime
// form (no prime square divides) and `data/nat/nat_square_part.ac` the square
// part decomposition.  This file connects squarefreeness to the number
// theory built on the prime factorisation: the support of the Mobius function
// (`mu(n) != 0`), the equality `omega(n) = Omega(n)` of the distinct and total
// prime factor counts, the squarefree/prime-power dichotomy, closure under the
// gcd, and the classical identity `sum_{d^2 | n} mu(d) = 1` at squarefree `n`
// and `0` otherwise (proved here in small cases).
from nat import Nat, mul_to_zero, mul_cancel_left, exp_ne_zero, divides_zero,
    divides_self, divides_lte, pos_of_ne_zero, lt_imp_lte_suc, lt_and_lte,
    lt_not_ref, lt_suc, lt_trans, lte_mul_both, lte_trans, gcd_divides_left,
    mul_assoc, mul_comm
from int import Int, add_neg, add_comm, mul_zero_right, mul_zero_left
from list import List, map, sum, product, is_permutation,
    list_contains_implies_count_geq_one, list_not_contains_impl_count_zero,
    unique_implies_no_duplicate, not_unique_implies_duplicate
from number_theory.factorisation import prime_factorisation, prime_factorisation_product,
    prime_factorisation_all_prime, prime_factorisation_unique, all_prime,
    all_prime_product_nonzero, all_prime_only_primes, count_prime_factor,
    count_prime_factor_mul, count_prime_factor_self, count_prime_factor_pow,
    prime_pow_divides_imp_count_le, count_le_imp_prime_pow_divides,
    divides_imp_count_prime_factor_le, prime_does_not_divide_one,
    prime_divisor_is_one_or_self
from number_theory.fermat import prime_divides_mul
from number_theory.mobius_inversion import nat_mobius, nat_mobius_zero,
    nat_mobius_one, nat_mobius_prime, nat_mobius_nonzero_iff_factorisation_unique,
    prime_factorisation_prime, permutation_of_unique_is_unique, unique_of_length_one
from number_theory.omega_omega_big import nat_omega, nat_prime_omega,
    nat_omega_eq_Omega_iff_factorisation_unique
from number_theory.liouville import nat_two_prime_local, nat_three_prime
from number_theory.divisor_sum import divisor_list, divisor_list_one,
    divisor_list_prime, divisor_list_zero, divisors_up_to, divisors_up_to_suc_yes,
    divisors_up_to_suc_no, divisors_up_to_one, one_divides_nat
from data.nat.nat_squarefree import is_squarefree, is_squarefree_intro,
    is_squarefree_apply, squarefree_of_divides, not_squarefree_of_square_divisor,
    one_is_squarefree
from data.nat.nat_square import is_square, is_square_intro
from data.nat.nat_squarefree_prime import no_prime_square_divides,
    no_prime_square_divides_intro, no_prime_square_divides_apply,
    squarefree_of_no_prime_square_divides
from data.nat.nat_square_part import square_divisor_pred
from data.list.list_filter_count import filter_cons_of_true, filter_cons_of_false
numerals Nat
numerals Int

// ---------------------------------------------------------------------------
// Squarefreeness and the prime factorisation
// ---------------------------------------------------------------------------

/// A positive squarefree natural has a squarefree (duplicate-free) prime
/// factorisation.
///
/// If some prime occurred twice, its square would divide the number, which
/// squarefreeness forbids.
theorem squarefree_imp_factorisation_unique(n: Nat) {
    Nat.0 < n and is_squarefree(n) implies prime_factorisation(n).is_unique
} by {
    if Nat.0 < n and is_squarefree(n) {
        Nat.1 <= n
        n != Nat.0
        if not prime_factorisation(n).is_unique {
            not_unique_implies_duplicate(prime_factorisation(n))
            let q: Nat satisfy { prime_factorisation(n).count(q) > Nat.1 }
            prime_factorisation_all_prime(n)
            all_prime(prime_factorisation(n))
            if not prime_factorisation(n).contains(q) {
                list_not_contains_impl_count_zero(prime_factorisation(n), q)
                prime_factorisation(n).count(q) = Nat.0
                Nat.1 < prime_factorisation(n).count(q)
                Nat.1 < Nat.0
                false
            }
            prime_factorisation(n).contains(q)
            all_prime_only_primes(prime_factorisation(n), q)
            q.is_prime
            prime_factorisation(n).count(q) = count_prime_factor(q, n)
            count_prime_factor(q, n) > Nat.1
            count_prime_factor(q, n) > Nat.1 = (Nat.1 < count_prime_factor(q, n))
            Nat.1 < count_prime_factor(q, n)
            lt_imp_lte_suc(Nat.1, count_prime_factor(q, n))
            Nat.1.suc <= count_prime_factor(q, n)
            Nat.1.suc = Nat.2
            Nat.2 <= count_prime_factor(q, n)
            count_le_imp_prime_pow_divides(q, Nat.2, n)
            q.pow(Nat.2).divides(n)
            q.pow(Nat.2) = q * q
            (q * q).divides(n)
            is_square_intro(q * q, q)
            is_square(q * q)
            if q * q = Nat.1 {
                q * q = Nat.1
                q.divides(Nat.1)
                prime_does_not_divide_one(q)
                not q.divides(Nat.1)
                false
            }
            q * q != Nat.1
            is_square(q * q) and (q * q).divides(n) and q * q != Nat.1
            not_squarefree_of_square_divisor(n, q * q)
            not is_squarefree(n)
            false
        }
        prime_factorisation(n).is_unique
    }
}

/// A positive natural whose prime factorisation has no repeated prime is
/// squarefree.
///
/// A square `p * p` dividing the number would force the multiplicity of `p`
/// to be at least two, contradicting uniqueness.
theorem factorisation_unique_imp_squarefree(n: Nat) {
    Nat.0 < n and prime_factorisation(n).is_unique implies is_squarefree(n)
} by {
    if Nat.0 < n and prime_factorisation(n).is_unique {
        n != Nat.0
        Nat.1 <= n
        forall(p: Nat) {
            if p.is_prime {
                Nat.1 < p
                p != Nat.0
                if (p * p).divides(n) {
                    mul_to_zero(p, p)
                    p * p != Nat.0
                    count_prime_factor_mul(p, p, p)
                    count_prime_factor(p, p * p) =
                        count_prime_factor(p, p) + count_prime_factor(p, p)
                    count_prime_factor_self(p)
                    count_prime_factor(p, p) = Nat.1
                    count_prime_factor(p, p * p) = Nat.1 + Nat.1
                    Nat.1 + Nat.1 = Nat.2
                    count_prime_factor(p, p * p) = Nat.2
                    divides_imp_count_prime_factor_le(p, p * p, n)
                    count_prime_factor(p, p * p) <= count_prime_factor(p, n)
                    Nat.2 <= count_prime_factor(p, n)
                    prime_factorisation(n).count(p) = count_prime_factor(p, n)
                    Nat.2 <= prime_factorisation(n).count(p)
                    unique_implies_no_duplicate(prime_factorisation(n), p)
                    prime_factorisation(n).count(p) <= Nat.1
                    lte_trans(Nat.2, prime_factorisation(n).count(p), Nat.1)
                    Nat.2 <= Nat.1
                    false
                }
                not (p * p).divides(n)
            }
            p.is_prime implies not (p * p).divides(n)
        }
        no_prime_square_divides_intro(n)
        no_prime_square_divides(n)
        squarefree_of_no_prime_square_divides(n)
        is_squarefree(n)
    }
}

/// A positive natural is squarefree exactly when its prime factorisation has
/// no repeated prime.
theorem is_squarefree_iff_factorisation_unique(n: Nat) {
    Nat.0 < n implies (is_squarefree(n) = prime_factorisation(n).is_unique)
} by {
    if Nat.0 < n {
        if prime_factorisation(n).is_unique {
            Nat.0 < n and prime_factorisation(n).is_unique
            if Nat.0 < n and prime_factorisation(n).is_unique {
                factorisation_unique_imp_squarefree(n)
                is_squarefree(n)
            }
        } else {
            not prime_factorisation(n).is_unique
            if is_squarefree(n) {
                Nat.0 < n and is_squarefree(n)
                if Nat.0 < n and is_squarefree(n) {
                    squarefree_imp_factorisation_unique(n)
                    prime_factorisation(n).is_unique
                }
                false
            }
            not is_squarefree(n)
        }
        is_squarefree(n) = prime_factorisation(n).is_unique
    }
}

/// Zero is not squarefree, since the square of two divides it.
theorem zero_not_squarefree {
    not is_squarefree(Nat.0)
} by {
    Nat.4 = Nat.2 * Nat.2
    is_square_intro(Nat.4, Nat.2)
    is_square(Nat.4)
    divides_zero(Nat.4)
    Nat.4.divides(Nat.0)
    Nat.4 != Nat.1
    is_square(Nat.4) and Nat.4.divides(Nat.0) and Nat.4 != Nat.1
    not_squarefree_of_square_divisor(Nat.0, Nat.4)
    not is_squarefree(Nat.0)
}

/// Squarefreeness is exactly the support of the Mobius function:
/// `is_squarefree(n) = (mu(n) != 0)` for every natural `n`.
///
/// At zero both sides are false; for positive `n` this is the classical
/// characterisation of squarefree numbers as those on which `mu` does not
/// vanish, read through the duplicate-free factorisation.
theorem is_squarefree_iff_mobius_nonzero(n: Nat) {
    is_squarefree(n) = (nat_mobius(n) != Int.0)
} by {
    if n = Nat.0 {
        zero_not_squarefree
        not is_squarefree(Nat.0)
        nat_mobius_zero
        nat_mobius(Nat.0) = Int.0
        not (nat_mobius(Nat.0) != Int.0)
        n = Nat.0
        is_squarefree(n) = is_squarefree(Nat.0)
        (nat_mobius(n) != Int.0) = (nat_mobius(Nat.0) != Int.0)
        is_squarefree(n) = (nat_mobius(n) != Int.0)
    } else {
        n != Nat.0
        pos_of_ne_zero(n)
        Nat.0 < n
        is_squarefree_iff_factorisation_unique(n)
        (is_squarefree(n) = prime_factorisation(n).is_unique)
        nat_mobius_nonzero_iff_factorisation_unique(n)
        (nat_mobius(n) != Int.0) = prime_factorisation(n).is_unique
        is_squarefree(n) = (nat_mobius(n) != Int.0)
    }
}

/// A positive natural is squarefree exactly when the number of its distinct
/// prime factors equals the number of its prime factors with multiplicity:
/// `is_squarefree(n) = (omega(n) = Omega(n))`.
///
/// This is the `omega(n) = Omega(n)` characterisation of squarefreeness,
/// expressed through the duplicate-free factorisation.
theorem is_squarefree_iff_omega_eq_Omega(n: Nat) {
    Nat.0 < n implies (is_squarefree(n) = (nat_omega(n) = nat_prime_omega(n)))
} by {
    if Nat.0 < n {
        is_squarefree_iff_factorisation_unique(n)
        (is_squarefree(n) = prime_factorisation(n).is_unique)
        nat_omega_eq_Omega_iff_factorisation_unique(n)
        (nat_omega(n) = nat_prime_omega(n)) = prime_factorisation(n).is_unique
        is_squarefree(n) = (nat_omega(n) = nat_prime_omega(n))
    }
}

// ---------------------------------------------------------------------------
// The counting function and the asymptotic density
// ---------------------------------------------------------------------------

/// The number of squarefree naturals at most `n`.
///
/// Since zero is not squarefree this counts the squarefree naturals in
/// `{1, ..., n}` as well.
define squarefree_count(n: Nat) -> Nat {
    match n {
        Nat.zero {
            Nat.0
        }
        Nat.suc(k) {
            if is_squarefree(k.suc) {
                squarefree_count(k) + Nat.1
            } else {
                squarefree_count(k)
            }
        }
    }
}

/// The squarefree count at zero is empty.
theorem squarefree_count_zero {
    squarefree_count(Nat.0) = Nat.0
}

/// Extending the range by one adds one exactly when the new top is squarefree.
theorem squarefree_count_step(n: Nat) {
    squarefree_count(n.suc) =
        if is_squarefree(n.suc) {
            squarefree_count(n) + Nat.1
        } else {
            squarefree_count(n)
        }
}

// The classical asymptotic: the number of squarefree numbers up to `n` grows
// like `6n / pi^2`, i.e. the squarefree numbers have natural density
// `6 / pi^2 ~ 0.6079`.  A full proof needs real analysis (expressing the
// count through `sum_{d^2 | k} mu(d) = |mu(k)|` and summing the floors
// `floor(n / d^2)`) and is left as a research target:
//
//   squarefree_count(n) / n  -->  6 / pi^2   as  n --> infinity.

// ---------------------------------------------------------------------------
// Squarefree and non-squarefree products
// ---------------------------------------------------------------------------

/// A prime is squarefree.
theorem prime_is_squarefree(p: Nat) {
    p.is_prime implies is_squarefree(p)
} by {
    if p.is_prime {
        prime_factorisation_prime(p)
        prime_factorisation(p) = List.singleton(p)
        List.singleton(p).length = Nat.1
        prime_factorisation(p).length = Nat.1
        unique_of_length_one(prime_factorisation(p))
        prime_factorisation(p).is_unique
        Nat.1 < p
        Nat.0 < Nat.1
        lt_and_lte(Nat.0, Nat.1, p)
        Nat.0 < p
        is_squarefree_iff_factorisation_unique(p)
        (is_squarefree(p) = prime_factorisation(p).is_unique)
        prime_factorisation(p).is_unique
        is_squarefree(p)
    }
}

/// A product of distinct primes is squarefree.
///
/// The list is its own (unique) prime factorisation up to permutation, so the
/// factorisation of the product has no repeated prime.
theorem product_of_unique_primes_squarefree(l: List[Nat]) {
    all_prime(l) and l.is_unique implies is_squarefree(product[Nat](l))
} by {
    if all_prime(l) and l.is_unique {
        all_prime_product_nonzero(l)
        product[Nat](l) != Nat.0
        pos_of_ne_zero(product[Nat](l))
        Nat.0 < product[Nat](l)
        lt_imp_lte_suc(Nat.0, product[Nat](l))
        Nat.0.suc <= product[Nat](l)
        Nat.0.suc = Nat.1
        Nat.1 <= product[Nat](l)
        prime_factorisation_product(product[Nat](l))
        product[Nat](prime_factorisation(product[Nat](l))) = product[Nat](l)
        prime_factorisation_all_prime(product[Nat](l))
        all_prime(prime_factorisation(product[Nat](l)))
        all_prime(l) and all_prime(prime_factorisation(product[Nat](l))) and
            product[Nat](l) = product[Nat](prime_factorisation(product[Nat](l)))
        if all_prime(l) and all_prime(prime_factorisation(product[Nat](l))) and
                product[Nat](l) = product[Nat](prime_factorisation(product[Nat](l))) {
            prime_factorisation_unique(l, prime_factorisation(product[Nat](l)))
            is_permutation(l, prime_factorisation(product[Nat](l)))
        }
        l.is_unique
        is_permutation(l, prime_factorisation(product[Nat](l))) and l.is_unique
        if is_permutation(l, prime_factorisation(product[Nat](l))) and l.is_unique {
            permutation_of_unique_is_unique(l, prime_factorisation(product[Nat](l)))
            prime_factorisation(product[Nat](l)).is_unique
        }
        is_squarefree_iff_factorisation_unique(product[Nat](l))
        (is_squarefree(product[Nat](l)) = prime_factorisation(product[Nat](l)).is_unique)
        prime_factorisation(product[Nat](l)).is_unique
        is_squarefree(product[Nat](l))
    }
}

/// A prime power with exponent at least two is not squarefree.
///
/// The square of the prime divides `p^k`, so `p^k` is divisible by a square
/// other than one.
theorem prime_pow_not_squarefree(p: Nat, k: Nat) {
    p.is_prime and Nat.2 <= k implies not is_squarefree(p.pow(k))
} by {
    if p.is_prime and Nat.2 <= k {
        Nat.1 < p
        p != Nat.0
        exp_ne_zero(p, k)
        p.pow(k) != Nat.0
        count_prime_factor_pow(p, k)
        count_prime_factor(p, p.pow(k)) = k
        Nat.2 <= count_prime_factor(p, p.pow(k))
        count_le_imp_prime_pow_divides(p, Nat.2, p.pow(k))
        p.pow(Nat.2).divides(p.pow(k))
        p.pow(Nat.2) = p * p
        (p * p).divides(p.pow(k))
        is_square_intro(p * p, p)
        is_square(p * p)
        if p * p = Nat.1 {
            p * p = Nat.1
            p.divides(Nat.1)
            prime_does_not_divide_one(p)
            not p.divides(Nat.1)
            false
        }
        p * p != Nat.1
        is_square(p * p) and (p * p).divides(p.pow(k)) and p * p != Nat.1
        not_squarefree_of_square_divisor(p.pow(k), p * p)
        not is_squarefree(p.pow(k))
    }
}

// ---------------------------------------------------------------------------
// Closure under the gcd
// ---------------------------------------------------------------------------

/// Squarefree numbers are closed under the gcd.
///
/// The gcd of two numbers divides each of them, and squarefreeness passes to
/// divisors.
theorem squarefree_gcd(a: Nat, b: Nat) {
    is_squarefree(a) and is_squarefree(b) implies is_squarefree(a.gcd(b))
} by {
    if is_squarefree(a) and is_squarefree(b) {
        gcd_divides_left(a, b)
        a.gcd(b).divides(a)
        is_squarefree(a) and a.gcd(b).divides(a)
        squarefree_of_divides(a, a.gcd(b))
        is_squarefree(a.gcd(b))
    }
}

// ---------------------------------------------------------------------------
// The classical identity  sum_{d^2 | n} mu(d) = |mu(n)|
// ---------------------------------------------------------------------------

/// The divisors `d` of `n` whose square divides `n`: the summation range of
/// the classical square-divisor Mobius identity.
define square_divisor_list(n: Nat) -> List[Nat] {
    divisor_list(n).filter(square_divisor_pred(n))
}

/// The sum of the Mobius values over the square divisors of `n`:
/// `sum_{d^2 | n} mu(d)`.
define mobius_square_divisor_sum(n: Nat) -> Int {
    sum(map(square_divisor_list(n), nat_mobius))
}

/// The value `sum_{d^2 | n} mu(d)` takes in the classical identity: `1` at
/// squarefree `n` and `0` otherwise.
define mobius_square_divisor_identity_value(n: Nat) -> Int {
    if is_squarefree(n) { Int.1 } else { Int.0 }
}

// The classical identity: `sum_{d^2 | n} mu(d)` is `1` when `n` is squarefree
// and `0` otherwise.  A full proof needs Mobius inversion over the square
// divisors (interchanging the divisor sum with the sum over `d`, or a pairing
// argument on the square divisors), and is left as a research target:
//
// theorem mobius_square_divisor_identity(n: Nat) {
//     Nat.0 < n implies
//         mobius_square_divisor_sum(n) = mobius_square_divisor_identity_value(n)
// }
//
// The identity is proved below in the small cases `n = 1`, `n = p` a prime,
// and `n = 4`.

/// Three does not divide four.
theorem three_not_divides_four {
    not Nat.3.divides(Nat.4)
} by {
    if Nat.3.divides(Nat.4) {
        Nat.4 = Nat.2 * Nat.2
        Nat.3.divides(Nat.2 * Nat.2)
        nat_three_prime
        Nat.3.is_prime
        prime_divides_mul(Nat.3, Nat.2, Nat.2)
        Nat.3.divides(Nat.2) or Nat.3.divides(Nat.2)
        Nat.3.divides(Nat.2)
        nat_two_prime_local
        Nat.2.is_prime
        prime_divisor_is_one_or_self(Nat.2, Nat.3)
        Nat.3 = Nat.1 or Nat.3 = Nat.2
        Nat.3 != Nat.1
        Nat.3 != Nat.2
        false
    }
}

/// Four does not divide one.
theorem four_not_divides_one {
    not Nat.4.divides(Nat.1)
} by {
    if Nat.4.divides(Nat.1) {
        let c: Nat satisfy { Nat.4 * c = Nat.1 }
        if c = Nat.0 {
            Nat.4 * Nat.0 = Nat.0
            Nat.0 = Nat.1
            false
        }
        c != Nat.0
        pos_of_ne_zero(c)
        Nat.0 < c
        lt_imp_lte_suc(Nat.0, c)
        Nat.0.suc <= c
        Nat.0.suc = Nat.1
        Nat.1 <= c
        lte_mul_both(Nat.4, Nat.1, c)
        Nat.4 * Nat.1 <= Nat.4 * c
        Nat.4 * Nat.1 = Nat.4
        Nat.4 <= Nat.4 * c
        Nat.4 * c = Nat.1
        Nat.4 <= Nat.1
        lt_suc(Nat.1)
        Nat.1 < Nat.2
        lt_suc(Nat.2)
        Nat.2 < Nat.3
        lt_suc(Nat.3)
        Nat.3 < Nat.4
        lt_trans(Nat.1, Nat.2, Nat.3)
        Nat.1 < Nat.3
        lt_trans(Nat.1, Nat.3, Nat.4)
        Nat.1 < Nat.4
        lt_and_lte(Nat.1, Nat.4, Nat.1)
        Nat.1 < Nat.1
        lt_not_ref(Nat.1)
        false
    }
}

/// The square of four does not divide four.
///
/// If `4 * 4` divided `4` then `4` would divide one, which is impossible.
theorem four_square_not_divides_four {
    not (Nat.4 * Nat.4).divides(Nat.4)
} by {
    if (Nat.4 * Nat.4).divides(Nat.4) {
        let c: Nat satisfy { Nat.4 * Nat.4 * c = Nat.4 }
        Nat.4 * Nat.4 * c = Nat.4
        Nat.4 * (Nat.4 * c) = Nat.4
        Nat.4 * Nat.1 = Nat.4
        Nat.4 * (Nat.4 * c) = Nat.4 * Nat.1
        Nat.4 != Nat.0
        mul_cancel_left(Nat.4, Nat.4 * c, Nat.1)
        Nat.4 * c = Nat.1
        Nat.4.divides(Nat.1)
        four_not_divides_one
        not Nat.4.divides(Nat.1)
        false
    }
}

/// The classical identity at `n = 1`.
///
/// The only square divisor of one is one, and `mu(1) = 1`; one is squarefree.
theorem mobius_square_divisor_identity_one {
    mobius_square_divisor_sum(Nat.1) = mobius_square_divisor_identity_value(Nat.1)
} by {
    divisor_list_one
    divisor_list(Nat.1) = List.cons(Nat.1, List.nil[Nat])
    square_divisor_pred(Nat.1)(Nat.1) = ((Nat.1 * Nat.1).divides(Nat.1))
    Nat.1 * Nat.1 = Nat.1
    divides_self(Nat.1)
    Nat.1.divides(Nat.1)
    (Nat.1 * Nat.1).divides(Nat.1)
    square_divisor_pred(Nat.1)(Nat.1)
    filter_cons_of_true(Nat.1, List.nil[Nat], square_divisor_pred(Nat.1))
    List.cons(Nat.1, List.nil[Nat]).filter(square_divisor_pred(Nat.1)) =
        List.cons(Nat.1, List.nil[Nat].filter(square_divisor_pred(Nat.1)))
    List.nil[Nat].filter(square_divisor_pred(Nat.1)) = List.nil[Nat]
    List.cons(Nat.1, List.nil[Nat]).filter(square_divisor_pred(Nat.1)) =
        List.cons(Nat.1, List.nil[Nat])
    square_divisor_list(Nat.1) = divisor_list(Nat.1).filter(square_divisor_pred(Nat.1))
    square_divisor_list(Nat.1) = List.cons(Nat.1, List.nil[Nat])
    map(List.cons(Nat.1, List.nil[Nat]), nat_mobius) =
        List.cons(nat_mobius(Nat.1), map(List.nil[Nat], nat_mobius))
    map(List.nil[Nat], nat_mobius) = List.nil[Int]
    map(List.cons(Nat.1, List.nil[Nat]), nat_mobius) =
        List.cons(nat_mobius(Nat.1), List.nil[Int])
    nat_mobius_one
    nat_mobius(Nat.1) = Int.1
    map(List.cons(Nat.1, List.nil[Nat]), nat_mobius) = List.cons(Int.1, List.nil[Int])
    sum(List.cons(Int.1, List.nil[Int])) = Int.1 + sum(List.nil[Int])
    sum(List.nil[Int]) = Int.0
    Int.1 + Int.0 = Int.1
    sum(map(List.cons(Nat.1, List.nil[Nat]), nat_mobius)) = Int.1
    sum(map(square_divisor_list(Nat.1), nat_mobius)) =
        sum(map(List.cons(Nat.1, List.nil[Nat]), nat_mobius))
    sum(map(square_divisor_list(Nat.1), nat_mobius)) = Int.1
    mobius_square_divisor_sum(Nat.1) = sum(map(square_divisor_list(Nat.1), nat_mobius))
    mobius_square_divisor_sum(Nat.1) = Int.1
    one_is_squarefree
    is_squarefree(Nat.1)
    mobius_square_divisor_identity_value(Nat.1) =
        if is_squarefree(Nat.1) { Int.1 } else { Int.0 }
        (if is_squarefree(Nat.1) { Int.1 } else { Int.0 }) = Int.1
    mobius_square_divisor_identity_value(Nat.1) = Int.1
    mobius_square_divisor_sum(Nat.1) = mobius_square_divisor_identity_value(Nat.1)
}

/// The classical identity at a prime `p`.
///
/// The square divisors of a prime are just `{1}`, so the sum is `mu(1) = 1`;
/// a prime is squarefree.
theorem mobius_square_divisor_identity_prime(p: Nat) {
    p.is_prime implies mobius_square_divisor_sum(p) = mobius_square_divisor_identity_value(p)
} by {
    if p.is_prime {
        divisor_list_prime(p)
        divisor_list(p) = List.cons(p, List.cons(Nat.1, List.nil[Nat]))
        Nat.1 < p
        p != Nat.0
        if (p * p).divides(p) {
            let c: Nat satisfy { p * p * c = p }
            p * p * c = p
            p * (p * c) = p
            p * Nat.1 = p
            p * (p * c) = p * Nat.1
            mul_cancel_left(p, p * c, Nat.1)
            p * c = Nat.1
            p.divides(Nat.1)
            prime_does_not_divide_one(p)
            not p.divides(Nat.1)
            false
        }
        not (p * p).divides(p)
        square_divisor_pred(p)(p) = ((p * p).divides(p))
        not square_divisor_pred(p)(p)
        filter_cons_of_false(p, List.cons(Nat.1, List.nil[Nat]), square_divisor_pred(p))
        List.cons(p, List.cons(Nat.1, List.nil[Nat])).filter(square_divisor_pred(p)) =
            List.cons(Nat.1, List.nil[Nat]).filter(square_divisor_pred(p))
        square_divisor_pred(p)(Nat.1) = ((Nat.1 * Nat.1).divides(p))
        Nat.1 * Nat.1 = Nat.1
        one_divides_nat(p)
        Nat.1.divides(p)
        (Nat.1 * Nat.1).divides(p)
        square_divisor_pred(p)(Nat.1)
        filter_cons_of_true(Nat.1, List.nil[Nat], square_divisor_pred(p))
        List.cons(Nat.1, List.nil[Nat]).filter(square_divisor_pred(p)) =
            List.cons(Nat.1, List.nil[Nat].filter(square_divisor_pred(p)))
        List.nil[Nat].filter(square_divisor_pred(p)) = List.nil[Nat]
        List.cons(Nat.1, List.nil[Nat]).filter(square_divisor_pred(p)) =
            List.cons(Nat.1, List.nil[Nat])
        List.cons(p, List.cons(Nat.1, List.nil[Nat])).filter(square_divisor_pred(p)) =
            List.cons(Nat.1, List.nil[Nat])
        square_divisor_list(p) = divisor_list(p).filter(square_divisor_pred(p))
        square_divisor_list(p) = List.cons(Nat.1, List.nil[Nat])
        map(List.cons(Nat.1, List.nil[Nat]), nat_mobius) =
            List.cons(nat_mobius(Nat.1), map(List.nil[Nat], nat_mobius))
        map(List.nil[Nat], nat_mobius) = List.nil[Int]
        map(List.cons(Nat.1, List.nil[Nat]), nat_mobius) =
            List.cons(nat_mobius(Nat.1), List.nil[Int])
        nat_mobius_one
        nat_mobius(Nat.1) = Int.1
        map(List.cons(Nat.1, List.nil[Nat]), nat_mobius) = List.cons(Int.1, List.nil[Int])
        sum(List.cons(Int.1, List.nil[Int])) = Int.1 + sum(List.nil[Int])
        sum(List.nil[Int]) = Int.0
        Int.1 + Int.0 = Int.1
        sum(map(List.cons(Nat.1, List.nil[Nat]), nat_mobius)) = Int.1
        sum(map(square_divisor_list(p), nat_mobius)) =
            sum(map(List.cons(Nat.1, List.nil[Nat]), nat_mobius))
        sum(map(square_divisor_list(p), nat_mobius)) = Int.1
        mobius_square_divisor_sum(p) = sum(map(square_divisor_list(p), nat_mobius))
        mobius_square_divisor_sum(p) = Int.1
        prime_is_squarefree(p)
        is_squarefree(p)
        mobius_square_divisor_identity_value(p) =
            if is_squarefree(p) { Int.1 } else { Int.0 }
        (if is_squarefree(p) { Int.1 } else { Int.0 }) = Int.1
        mobius_square_divisor_identity_value(p) = Int.1
        mobius_square_divisor_sum(p) = mobius_square_divisor_identity_value(p)
    }
}

/// The classical identity at `n = 4`.
///
/// The square divisors of `4 = 2^2` are `{1, 2}`, so the sum is
/// `mu(1) + mu(2) = 1 - 1 = 0`; four is not squarefree.  This is the small
/// case exhibiting the vanishing branch of the identity.
theorem mobius_square_divisor_identity_four {
    mobius_square_divisor_sum(Nat.4) = mobius_square_divisor_identity_value(Nat.4)
} by {
    divides_self(Nat.4)
    Nat.4.divides(Nat.4)
    divisors_up_to_suc_yes(Nat.4, Nat.3)
    divisors_up_to(Nat.4, Nat.3.suc) = List.cons(Nat.3.suc, divisors_up_to(Nat.4, Nat.3))
    Nat.3.suc = Nat.4
    divisors_up_to(Nat.4, Nat.4) = List.cons(Nat.4, divisors_up_to(Nat.4, Nat.3))
    three_not_divides_four
    not Nat.3.divides(Nat.4)
    divisors_up_to_suc_no(Nat.4, Nat.2)
    divisors_up_to(Nat.4, Nat.2.suc) = divisors_up_to(Nat.4, Nat.2)
    Nat.2.suc = Nat.3
    divisors_up_to(Nat.4, Nat.3) = divisors_up_to(Nat.4, Nat.2)
    divisors_up_to(Nat.4, Nat.4) = List.cons(Nat.4, divisors_up_to(Nat.4, Nat.2))
    Nat.2 * Nat.2 = Nat.4
    Nat.2.divides(Nat.4)
    divisors_up_to_suc_yes(Nat.4, Nat.1)
    divisors_up_to(Nat.4, Nat.1.suc) = List.cons(Nat.1.suc, divisors_up_to(Nat.4, Nat.1))
    Nat.1.suc = Nat.2
    divisors_up_to(Nat.4, Nat.2) = List.cons(Nat.2, divisors_up_to(Nat.4, Nat.1))
    divisors_up_to_one(Nat.4)
    divisors_up_to(Nat.4, Nat.1) = List.cons(Nat.1, List.nil[Nat])
    divisors_up_to(Nat.4, Nat.2) =
        List.cons(Nat.2, List.cons(Nat.1, List.nil[Nat]))
    divisors_up_to(Nat.4, Nat.4) =
        List.cons(Nat.4, List.cons(Nat.2, List.cons(Nat.1, List.nil[Nat])))
    divisor_list(Nat.4) = divisors_up_to(Nat.4, Nat.4)
    divisor_list(Nat.4) =
        List.cons(Nat.4, List.cons(Nat.2, List.cons(Nat.1, List.nil[Nat])))
    four_square_not_divides_four
    not (Nat.4 * Nat.4).divides(Nat.4)
    square_divisor_pred(Nat.4)(Nat.4) = ((Nat.4 * Nat.4).divides(Nat.4))
    not square_divisor_pred(Nat.4)(Nat.4)
    filter_cons_of_false(Nat.4,
        List.cons(Nat.2, List.cons(Nat.1, List.nil[Nat])), square_divisor_pred(Nat.4))
    List.cons(Nat.4, List.cons(Nat.2, List.cons(Nat.1, List.nil[Nat]))).filter(square_divisor_pred(Nat.4)) =
        List.cons(Nat.2, List.cons(Nat.1, List.nil[Nat])).filter(square_divisor_pred(Nat.4))
    Nat.2 * Nat.2 = Nat.4
    divides_self(Nat.4)
    Nat.4.divides(Nat.4)
    (Nat.2 * Nat.2).divides(Nat.4)
    square_divisor_pred(Nat.4)(Nat.2) = ((Nat.2 * Nat.2).divides(Nat.4))
    square_divisor_pred(Nat.4)(Nat.2)
    filter_cons_of_true(Nat.2, List.cons(Nat.1, List.nil[Nat]), square_divisor_pred(Nat.4))
    List.cons(Nat.2, List.cons(Nat.1, List.nil[Nat])).filter(square_divisor_pred(Nat.4)) =
        List.cons(Nat.2,
            List.cons(Nat.1, List.nil[Nat]).filter(square_divisor_pred(Nat.4)))
    Nat.1 * Nat.1 = Nat.1
    one_divides_nat(Nat.4)
    Nat.1.divides(Nat.4)
    (Nat.1 * Nat.1).divides(Nat.4)
    square_divisor_pred(Nat.4)(Nat.1) = ((Nat.1 * Nat.1).divides(Nat.4))
    square_divisor_pred(Nat.4)(Nat.1)
    filter_cons_of_true(Nat.1, List.nil[Nat], square_divisor_pred(Nat.4))
    List.cons(Nat.1, List.nil[Nat]).filter(square_divisor_pred(Nat.4)) =
        List.cons(Nat.1, List.nil[Nat].filter(square_divisor_pred(Nat.4)))
    List.nil[Nat].filter(square_divisor_pred(Nat.4)) = List.nil[Nat]
    List.cons(Nat.1, List.nil[Nat]).filter(square_divisor_pred(Nat.4)) =
        List.cons(Nat.1, List.nil[Nat])
    List.cons(Nat.2, List.cons(Nat.1, List.nil[Nat])).filter(square_divisor_pred(Nat.4)) =
        List.cons(Nat.2, List.cons(Nat.1, List.nil[Nat]))
    List.cons(Nat.4, List.cons(Nat.2, List.cons(Nat.1, List.nil[Nat]))).filter(square_divisor_pred(Nat.4)) =
        List.cons(Nat.2, List.cons(Nat.1, List.nil[Nat]))
    square_divisor_list(Nat.4) = divisor_list(Nat.4).filter(square_divisor_pred(Nat.4))
    square_divisor_list(Nat.4) = List.cons(Nat.2, List.cons(Nat.1, List.nil[Nat]))
    map(List.cons(Nat.2, List.cons(Nat.1, List.nil[Nat])), nat_mobius) =
        List.cons(nat_mobius(Nat.2),
            map(List.cons(Nat.1, List.nil[Nat]), nat_mobius))
    map(List.cons(Nat.1, List.nil[Nat]), nat_mobius) =
        List.cons(nat_mobius(Nat.1), map(List.nil[Nat], nat_mobius))
    map(List.nil[Nat], nat_mobius) = List.nil[Int]
    nat_two_prime_local
    Nat.2.is_prime
    nat_mobius_prime(Nat.2)
    nat_mobius(Nat.2) = -Int.1
    nat_mobius_one
    nat_mobius(Nat.1) = Int.1
    map(List.cons(Nat.2, List.cons(Nat.1, List.nil[Nat])), nat_mobius) =
        List.cons(-Int.1, List.cons(Int.1, List.nil[Int]))
    sum(List.cons(-Int.1, List.cons(Int.1, List.nil[Int]))) =
        -Int.1 + sum(List.cons(Int.1, List.nil[Int]))
    sum(List.cons(Int.1, List.nil[Int])) = Int.1 + sum(List.nil[Int])
    sum(List.nil[Int]) = Int.0
    Int.1 + Int.0 = Int.1
    sum(List.cons(Int.1, List.nil[Int])) = Int.1
    sum(List.cons(-Int.1, List.cons(Int.1, List.nil[Int]))) = -Int.1 + Int.1
    add_comm(Int.1, -Int.1)
    Int.1 + -Int.1 = -Int.1 + Int.1
    add_neg(Int.1)
    Int.1 + -Int.1 = Int.0
    -Int.1 + Int.1 = Int.0
    sum(List.cons(-Int.1, List.cons(Int.1, List.nil[Int]))) = Int.0
    sum(map(List.cons(Nat.2, List.cons(Nat.1, List.nil[Nat])), nat_mobius)) = Int.0
    sum(map(square_divisor_list(Nat.4), nat_mobius)) =
        sum(map(List.cons(Nat.2, List.cons(Nat.1, List.nil[Nat])), nat_mobius))
    sum(map(square_divisor_list(Nat.4), nat_mobius)) = Int.0
    mobius_square_divisor_sum(Nat.4) = sum(map(square_divisor_list(Nat.4), nat_mobius))
    mobius_square_divisor_sum(Nat.4) = Int.0
    Nat.4 = Nat.2 * Nat.2
    is_square_intro(Nat.4, Nat.2)
    is_square(Nat.4)
    divides_self(Nat.4)
    Nat.4.divides(Nat.4)
    Nat.4 != Nat.1
    is_square(Nat.4) and Nat.4.divides(Nat.4) and Nat.4 != Nat.1
    not_squarefree_of_square_divisor(Nat.4, Nat.4)
    not is_squarefree(Nat.4)
    mobius_square_divisor_identity_value(Nat.4) =
        if is_squarefree(Nat.4) { Int.1 } else { Int.0 }
        (if is_squarefree(Nat.4) { Int.1 } else { Int.0 }) = Int.0
    mobius_square_divisor_identity_value(Nat.4) = Int.0
    mobius_square_divisor_sum(Nat.4) = mobius_square_divisor_identity_value(Nat.4)
}
