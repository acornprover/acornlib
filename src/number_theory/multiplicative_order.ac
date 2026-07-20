from number_theory.totient import Nat, euler, coprime_residues, coprime_residues_contains_iff,
    coprime_residues_length
from nat import has_min, is_min, false_below, gcd_zero_left
from number_theory.congruence import congr_mod_refl, congr_mod_mul, congr_mod_trans, congr_mod_symm,
    congr_mod_pow, mod_add_mul
from nat import exp_add, exp_mul, one_exp, pow_distrib_mul
from nat import division_theorem, lt_not_ref, lte_imp_not_lt, small_mod, add_sub,
    lt_add_left, lt_and_lte, trichotomy, divides_lte, gcd_divides_left,
    gcd_divides_right, cofactor, divides_cancel_right, div_mul, mul_to_zero,
    divides_gcd, divides_symm
from number_theory.coprime import coprime_pow_right, coprime_divides_of_divides_mul,
    coprime_comm, coprime_mul
from number_theory.crt import nat_coprime_combine
from number_theory.factorisation import coprime_factors_mul_lcm
from list import List
numerals Nat

/// The positive exponents at which `a` is congruent to `1` modulo `n`.
define multiplicative_order_witness(a: Nat, n: Nat) -> (Nat -> Bool) {
    function(k: Nat) {
        Nat.0 < k and a.pow(k).congr_mod(Nat.1, n)
    }
}

/// Applying the witness predicate unfolds to positivity plus the defining congruence.
theorem multiplicative_order_witness_apply(a: Nat, n: Nat, k: Nat) {
    multiplicative_order_witness(a, n)(k) =
        (Nat.0 < k and a.pow(k).congr_mod(Nat.1, n))
}

/// True when `k` is the least positive exponent with `a^k ≡ 1 (mod n)`.
define is_multiplicative_order_mod(a: Nat, n: Nat, k: Nat) -> Bool {
    is_min(multiplicative_order_witness(a, n), k)
}

/// Multiplicative-order membership unfolds to the minimum predicate.
theorem is_multiplicative_order_mod_unfold(a: Nat, n: Nat, k: Nat) {
    is_multiplicative_order_mod(a, n, k) =
        is_min(multiplicative_order_witness(a, n), k)
}

/// Euler's totient is positive for every positive modulus.
theorem totient_positive(n: Nat) {
    n != Nat.0 implies Nat.0 < n.totient
} by {
    if n != Nat.0 {
        let residues: List[Nat] = coprime_residues(n)
        if n = Nat.1 {
            gcd_zero_left(Nat.1)
            Nat.0.gcd(Nat.1) = Nat.1
            Nat.0.coprime(n)
            Nat.0 < n
            coprime_residues_contains_iff(n, Nat.0)
            residues.contains(Nat.0)
            match residues {
                List.nil {
                    false
                }
                List.cons(head, tail) {
                    residues.length = tail.length.suc
                    Nat.0 < residues.length
                }
            }
        } else {
            if n < Nat.1 {
                n = Nat.0
                false
            }
            Nat.1 < n
            Nat.1.coprime(n)
            coprime_residues_contains_iff(n, Nat.1)
            residues.contains(Nat.1)
            match residues {
                List.nil {
                    false
                }
                List.cons(head, tail) {
                    residues.length = tail.length.suc
                    Nat.0 < residues.length
                }
            }
        }
        coprime_residues_length(n)
        residues.length = n.totient
        Nat.0 < n.totient
    }
}

/// The multiplicative order of `a` modulo `n`, with placeholder value `0`
/// outside the positive-coprime domain.
let multiplicative_order_mod(a: Nat, n: Nat) -> k: Nat satisfy {
    (n != Nat.0 and a.coprime(n) and is_multiplicative_order_mod(a, n, k))
        or ((n = Nat.0 or not a.coprime(n)) and k = Nat.0)
} by {
    if n != Nat.0 and a.coprime(n) {
        euler(n, a)
        a.pow(n.totient).congr_mod(Nat.1, n)
        totient_positive(n)
        Nat.0 < n.totient
        a.pow(n.totient).congr_mod(Nat.1, n)
        multiplicative_order_witness(a, n)(n.totient) =
            (Nat.0 < n.totient and a.pow(n.totient).congr_mod(Nat.1, n))
        Nat.0 < n.totient and a.pow(n.totient).congr_mod(Nat.1, n)
        multiplicative_order_witness(a, n)(n.totient)
        has_min(multiplicative_order_witness(a, n), n.totient)
        let m: Nat satisfy { is_min(multiplicative_order_witness(a, n), m) }
        is_min(multiplicative_order_witness(a, n), m)
        is_multiplicative_order_mod(a, n, m)
        n != Nat.0 and a.coprime(n) and is_multiplicative_order_mod(a, n, m)
        (n != Nat.0 and a.coprime(n) and is_multiplicative_order_mod(a, n, m)) or ((n = Nat.0 or not a.coprime(n)) and m = Nat.0)
    } else {
        Nat.0 = Nat.0
        n = Nat.0 or not a.coprime(n)
        (n = Nat.0 or not a.coprime(n)) and Nat.0 = Nat.0
        (n != Nat.0 and a.coprime(n) and is_multiplicative_order_mod(a, n, Nat.0)) or ((n = Nat.0 or not a.coprime(n)) and Nat.0 = Nat.0)
    }
}

/// Characterisation of the selected order value, including its placeholder branch.
theorem multiplicative_order_mod_spec(a: Nat, n: Nat) {
    (n != Nat.0 and a.coprime(n) and
        is_multiplicative_order_mod(a, n, multiplicative_order_mod(a, n)))
        or ((n = Nat.0 or not a.coprime(n)) and multiplicative_order_mod(a, n) = Nat.0)
}

/// On the positive-coprime domain, the selected order is an order witness.
theorem multiplicative_order_mod_is_order(a: Nat, n: Nat) {
    n != Nat.0 and a.coprime(n)
        implies is_multiplicative_order_mod(a, n, multiplicative_order_mod(a, n))
} by {
    if n != Nat.0 and a.coprime(n) {
        multiplicative_order_mod_spec(a, n)
        if n != Nat.0 and a.coprime(n) and
            is_multiplicative_order_mod(a, n, multiplicative_order_mod(a, n)) {
        } else {
            (n = Nat.0 or not a.coprime(n)) and multiplicative_order_mod(a, n) = Nat.0
            false
        }
    }
}

/// Any multiplicative order modulo `n` is positive.
theorem multiplicative_order_positive(a: Nat, n: Nat, k: Nat) {
    is_multiplicative_order_mod(a, n, k) implies Nat.0 < k
} by {
    if is_multiplicative_order_mod(a, n, k) {
        is_min(multiplicative_order_witness(a, n), k) =
            is_multiplicative_order_mod(a, n, k)
        is_min(multiplicative_order_witness(a, n), k)
        is_min(multiplicative_order_witness(a, n), k) =
            (multiplicative_order_witness(a, n)(k) and
                false_below(multiplicative_order_witness(a, n), k))
        multiplicative_order_witness(a, n)(k) and
            false_below(multiplicative_order_witness(a, n), k)
        multiplicative_order_witness(a, n)(k)
        multiplicative_order_witness(a, n)(k) =
            (Nat.0 < k and a.pow(k).congr_mod(Nat.1, n))
        Nat.0 < k and a.pow(k).congr_mod(Nat.1, n)
        Nat.0 < k
    }
}

/// Any multiplicative order modulo `n` satisfies the defining congruence.
theorem multiplicative_order_pow_congr_one(a: Nat, n: Nat, k: Nat) {
    is_multiplicative_order_mod(a, n, k) implies a.pow(k).congr_mod(Nat.1, n)
} by {
    if is_multiplicative_order_mod(a, n, k) {
        is_min(multiplicative_order_witness(a, n), k) =
            is_multiplicative_order_mod(a, n, k)
        is_min(multiplicative_order_witness(a, n), k)
        is_min(multiplicative_order_witness(a, n), k) =
            (multiplicative_order_witness(a, n)(k) and
                false_below(multiplicative_order_witness(a, n), k))
        multiplicative_order_witness(a, n)(k) and
            false_below(multiplicative_order_witness(a, n), k)
        multiplicative_order_witness(a, n)(k)
        multiplicative_order_witness(a, n)(k) =
            (Nat.0 < k and a.pow(k).congr_mod(Nat.1, n))
        Nat.0 < k and a.pow(k).congr_mod(Nat.1, n)
        a.pow(k).congr_mod(Nat.1, n)
    }
}

/// No smaller positive exponent can satisfy `a^j ≡ 1 (mod n)`.
theorem multiplicative_order_minimal(a: Nat, n: Nat, k: Nat, j: Nat) {
    is_multiplicative_order_mod(a, n, k)
        and Nat.0 < j and a.pow(j).congr_mod(Nat.1, n)
        implies k <= j
} by {
    if is_multiplicative_order_mod(a, n, k)
        and Nat.0 < j and a.pow(j).congr_mod(Nat.1, n) {
        let f = multiplicative_order_witness(a, n)
        is_min(f, k) = is_multiplicative_order_mod(a, n, k)
        is_min(f, k)
        is_min(f, k) = (f(k) and false_below(f, k))
        f(k) and false_below(f, k)
        false_below(f, k)
        f(j) = (Nat.0 < j and a.pow(j).congr_mod(Nat.1, n))
        f(j)
        if j < k {
            false_below(f, k) = forall(x: Nat) {
                x < k implies not f(x)
            }
            not f(j)
            false
        } else {
            k <= j
        }
        k <= j
    }
}

/// The selected `multiplicative_order_mod` is positive on the positive-coprime domain.
theorem multiplicative_order_mod_positive(a: Nat, n: Nat) {
    n != Nat.0 and a.coprime(n) implies Nat.0 < multiplicative_order_mod(a, n)
} by {
    if n != Nat.0 and a.coprime(n) {
        multiplicative_order_mod_is_order(a, n)
        is_multiplicative_order_mod(a, n, multiplicative_order_mod(a, n))
        multiplicative_order_positive(a, n, multiplicative_order_mod(a, n))
    }
}

/// An explicit multiplicative order equals the selected order on the
/// positive-coprime domain.
theorem multiplicative_order_mod_eq_of_is_order(a: Nat, n: Nat, k: Nat) {
    n != Nat.0 and a.coprime(n) and is_multiplicative_order_mod(a, n, k)
        implies multiplicative_order_mod(a, n) = k
} by {
    if n != Nat.0 and a.coprime(n) and is_multiplicative_order_mod(a, n, k) {
        multiplicative_order_mod_is_order(a, n)
        is_multiplicative_order_mod(a, n, multiplicative_order_mod(a, n))
        multiplicative_order_positive(a, n, k)
        Nat.0 < k
        multiplicative_order_pow_congr_one(a, n, k)
        a.pow(k).congr_mod(Nat.1, n)
        multiplicative_order_minimal(a, n, multiplicative_order_mod(a, n), k)
        multiplicative_order_mod(a, n) <= k
        multiplicative_order_mod_positive(a, n)
        Nat.0 < multiplicative_order_mod(a, n)
        multiplicative_order_pow_congr_one(
            a, n, multiplicative_order_mod(a, n))
        a.pow(multiplicative_order_mod(a, n)).congr_mod(Nat.1, n)
        multiplicative_order_minimal(a, n, k, multiplicative_order_mod(a, n))
        k <= multiplicative_order_mod(a, n)
        multiplicative_order_mod(a, n) = k
    }
}

/// On the positive-coprime domain, an exponent is the multiplicative order
/// exactly when it equals the selected order.
theorem is_multiplicative_order_mod_iff_eq_selected(a: Nat, n: Nat, k: Nat) {
    n != Nat.0 and a.coprime(n) implies
        (is_multiplicative_order_mod(a, n, k) =
            (multiplicative_order_mod(a, n) = k))
} by {
    if n != Nat.0 and a.coprime(n) {
        if is_multiplicative_order_mod(a, n, k) {
            multiplicative_order_mod_eq_of_is_order(a, n, k)
            multiplicative_order_mod(a, n) = k
        }
        if multiplicative_order_mod(a, n) = k {
            multiplicative_order_mod_is_order(a, n)
            is_multiplicative_order_mod(a, n, multiplicative_order_mod(a, n))
            is_multiplicative_order_mod(a, n, k)
        }
        (is_multiplicative_order_mod(a, n, k) =
            (multiplicative_order_mod(a, n) = k)) = true
    }
}

/// The selected multiplicative order is nonzero on the positive-coprime
/// domain.
theorem multiplicative_order_mod_ne_zero(a: Nat, n: Nat) {
    n != Nat.0 and a.coprime(n) implies multiplicative_order_mod(a, n) != Nat.0
} by {
    if n != Nat.0 and a.coprime(n) {
        multiplicative_order_mod_positive(a, n)
        Nat.0 < multiplicative_order_mod(a, n)
        if multiplicative_order_mod(a, n) = Nat.0 {
            Nat.0 < Nat.0
            lt_not_ref(Nat.0)
            false
        }
    }
}

/// The selected multiplicative order is zero for the zero modulus.
theorem multiplicative_order_mod_zero_modulus(a: Nat) {
    multiplicative_order_mod(a, Nat.0) = Nat.0
} by {
    multiplicative_order_mod_spec(a, Nat.0)
    if Nat.0 != Nat.0 and a.coprime(Nat.0)
        and is_multiplicative_order_mod(a, Nat.0, multiplicative_order_mod(a, Nat.0)) {
        Nat.0 != Nat.0
        false
    } else {
        (Nat.0 = Nat.0 or not a.coprime(Nat.0)) and multiplicative_order_mod(a, Nat.0) = Nat.0
    }
}

/// The selected multiplicative order is zero outside the coprime domain.
theorem multiplicative_order_mod_not_coprime(a: Nat, n: Nat) {
    not a.coprime(n) implies multiplicative_order_mod(a, n) = Nat.0
} by {
    if not a.coprime(n) {
        multiplicative_order_mod_spec(a, n)
        if n != Nat.0 and a.coprime(n)
            and is_multiplicative_order_mod(a, n, multiplicative_order_mod(a, n)) {
            false
        } else {
            (n = Nat.0 or not a.coprime(n)) and multiplicative_order_mod(a, n) = Nat.0
        }
    }
}

/// If the selected multiplicative order is nonzero, the modulus is nonzero
/// and the base is coprime to it.
theorem multiplicative_order_mod_ne_zero_imp_domain(a: Nat, n: Nat) {
    multiplicative_order_mod(a, n) != Nat.0 implies n != Nat.0 and a.coprime(n)
} by {
    if multiplicative_order_mod(a, n) != Nat.0 {
        if n = Nat.0 {
            multiplicative_order_mod_zero_modulus(a)
            false
        }
        n != Nat.0
        if not a.coprime(n) {
            multiplicative_order_mod_not_coprime(a, n)
            false
        }
        a.coprime(n)
        n != Nat.0 and a.coprime(n)
    }
}

/// The selected multiplicative order is nonzero exactly on the
/// positive-coprime domain.
theorem multiplicative_order_mod_ne_zero_iff_domain(a: Nat, n: Nat) {
    (multiplicative_order_mod(a, n) != Nat.0) = (n != Nat.0 and a.coprime(n))
} by {
    if multiplicative_order_mod(a, n) != Nat.0 {
        multiplicative_order_mod_ne_zero_imp_domain(a, n)
        n != Nat.0 and a.coprime(n)
    }
    if n != Nat.0 and a.coprime(n) {
        multiplicative_order_mod_ne_zero(a, n)
        multiplicative_order_mod(a, n) != Nat.0
    }
    if not ((multiplicative_order_mod(a, n) != Nat.0) =
        (n != Nat.0 and a.coprime(n))) {
        false
    }
    ((multiplicative_order_mod(a, n) != Nat.0) =
        (n != Nat.0 and a.coprime(n))) = true
}

/// The selected `multiplicative_order_mod` satisfies `a^ord ≡ 1` on its domain.
theorem multiplicative_order_mod_pow_congr_one(a: Nat, n: Nat) {
    n != Nat.0 and a.coprime(n)
        implies a.pow(multiplicative_order_mod(a, n)).congr_mod(Nat.1, n)
} by {
    if n != Nat.0 and a.coprime(n) {
        multiplicative_order_mod_is_order(a, n)
        is_multiplicative_order_mod(a, n, multiplicative_order_mod(a, n))
        multiplicative_order_pow_congr_one(a, n, multiplicative_order_mod(a, n))
    }
}

/// The selected order is minimal among positive exponents giving congruence to `1`.
theorem multiplicative_order_mod_minimal(a: Nat, n: Nat, j: Nat) {
    n != Nat.0 and a.coprime(n) and Nat.0 < j and a.pow(j).congr_mod(Nat.1, n)
        implies multiplicative_order_mod(a, n) <= j
} by {
    if n != Nat.0 and a.coprime(n) and Nat.0 < j and a.pow(j).congr_mod(Nat.1, n) {
        let ord: Nat = multiplicative_order_mod(a, n)
        multiplicative_order_mod_is_order(a, n)
        is_multiplicative_order_mod(a, n, multiplicative_order_mod(a, n))
        is_multiplicative_order_mod(a, n, ord)
        multiplicative_order_minimal(a, n, ord, j)
        ord <= j
        ord = multiplicative_order_mod(a, n)
        multiplicative_order_mod(a, n) <= j
    }
}

/// If `a^e ≡ 1`, then appending a multiple of an order exponent leaves
/// `a^r` congruent to `a^(r+e)`.
theorem pow_congr_add_order_exponent(a: Nat, n: Nat, e: Nat, r: Nat) {
    a.pow(e).congr_mod(Nat.1, n) implies a.pow(r + e).congr_mod(a.pow(r), n)
} by {
    if a.pow(e).congr_mod(Nat.1, n) {
        exp_add(a, r, e)
        a.pow(r + e) = a.pow(r) * a.pow(e)
        congr_mod_refl(a.pow(r), n)
        a.pow(r).congr_mod(a.pow(r), n)
        congr_mod_mul(a.pow(r), a.pow(e), a.pow(r), Nat.1, n)
        (a.pow(r) * a.pow(e)).congr_mod(a.pow(r) * Nat.1, n)
        a.pow(r) * Nat.1 = a.pow(r)
        a.pow(r + e).congr_mod(a.pow(r), n)
    }
}

/// Powers whose exponents differ by adding the multiplicative order are congruent.
theorem pow_congr_add_multiplicative_order_mod(a: Nat, n: Nat, r: Nat) {
    n != Nat.0 and a.coprime(n)
        implies a.pow(r + multiplicative_order_mod(a, n)).congr_mod(a.pow(r), n)
} by {
    if n != Nat.0 and a.coprime(n) {
        multiplicative_order_mod_pow_congr_one(a, n)
        pow_congr_add_order_exponent(a, n, multiplicative_order_mod(a, n), r)
    }
}

/// If `a^e ≡ 1`, then every multiple of `e` is also an exponent giving
/// congruence to `1`.
theorem pow_mul_order_exponent_congr_one(a: Nat, n: Nat, e: Nat, q: Nat) {
    a.pow(e).congr_mod(Nat.1, n) implies a.pow(q * e).congr_mod(Nat.1, n)
} by {
    if a.pow(e).congr_mod(Nat.1, n) {
        congr_mod_pow(a.pow(e), Nat.1, n, q)
        a.pow(e).pow(q).congr_mod(Nat.1.pow(q), n)
        exp_mul(a, e, q)
        a.pow(e * q) = a.pow(e).pow(q)
        e * q = q * e
        a.pow(q * e) = a.pow(e).pow(q)
        one_exp(q)
        Nat.1.pow(q) = Nat.1
        a.pow(q * e).congr_mod(Nat.1, n)
    }
}

/// Splitting an exponent into `q * e + r`: if `a^e ≡ 1`, then
/// `a^(q*e+r) ≡ a^r` modulo `n`.
theorem pow_congr_add_order_multiple(a: Nat, n: Nat, e: Nat, q: Nat, r: Nat) {
    a.pow(e).congr_mod(Nat.1, n) implies a.pow(q * e + r).congr_mod(a.pow(r), n)
} by {
    if a.pow(e).congr_mod(Nat.1, n) {
        pow_mul_order_exponent_congr_one(a, n, e, q)
        a.pow(q * e).congr_mod(Nat.1, n)
        exp_add(a, q * e, r)
        a.pow(q * e + r) = a.pow(q * e) * a.pow(r)
        congr_mod_refl(a.pow(r), n)
        a.pow(r).congr_mod(a.pow(r), n)
        congr_mod_mul(a.pow(q * e), a.pow(r), Nat.1, a.pow(r), n)
        (a.pow(q * e) * a.pow(r)).congr_mod(Nat.1 * a.pow(r), n)
        Nat.1 * a.pow(r) = a.pow(r)
        a.pow(q * e + r).congr_mod(a.pow(r), n)
    }
}

/// Reducing an exponent modulo a positive exponent `e` with `a^e ≡ 1`
/// preserves the power modulo `n`.
theorem pow_congr_mod_order_exponent(a: Nat, n: Nat, e: Nat, m: Nat) {
    Nat.0 < e and a.pow(e).congr_mod(Nat.1, n)
        implies a.pow(m).congr_mod(a.pow(m.mod(e)), n)
} by {
    if Nat.0 < e and a.pow(e).congr_mod(Nat.1, n) {
        division_theorem(m, e)
        let (q: Nat, r: Nat) satisfy {
            r < e and m = q * e + r
        }
        pow_congr_add_order_multiple(a, n, e, q, r)
        a.pow(q * e + r).congr_mod(a.pow(r), n)
        a.pow(m).congr_mod(a.pow(r), n)
        mod_add_mul(q, e, r)
        (q * e + r).mod(e) = r.mod(e)
        m.mod(e) = r.mod(e)
        small_mod(r, e)
        r.mod(e) = r
        m.mod(e) = r
        a.pow(m.mod(e)) = a.pow(r)
        a.pow(m).congr_mod(a.pow(m.mod(e)), n)
    }
}

/// Reducing an exponent modulo a multiplicative order preserves the power.
theorem pow_congr_mod_multiplicative_order(a: Nat, n: Nat, ord: Nat, m: Nat) {
    is_multiplicative_order_mod(a, n, ord)
        implies a.pow(m).congr_mod(a.pow(m.mod(ord)), n)
} by {
    if is_multiplicative_order_mod(a, n, ord) {
        multiplicative_order_positive(a, n, ord)
        multiplicative_order_pow_congr_one(a, n, ord)
        Nat.0 < ord and a.pow(ord).congr_mod(Nat.1, n)
        pow_congr_mod_order_exponent(a, n, ord, m)
        a.pow(m).congr_mod(a.pow(m.mod(ord)), n)
    }
}

/// Reducing an exponent modulo the selected multiplicative order preserves the power.
theorem pow_congr_mod_multiplicative_order_mod(a: Nat, n: Nat, m: Nat) {
    n != Nat.0 and a.coprime(n)
        implies a.pow(m).congr_mod(a.pow(m.mod(multiplicative_order_mod(a, n))), n)
} by {
    if n != Nat.0 and a.coprime(n) {
        multiplicative_order_mod_is_order(a, n)
        is_multiplicative_order_mod(a, n, multiplicative_order_mod(a, n))
        pow_congr_mod_multiplicative_order(a, n, multiplicative_order_mod(a, n), m)
    }
}

/// The order divides every exponent whose power is congruent to `1`.
theorem multiplicative_order_divides_exponent(a: Nat, n: Nat, ord: Nat, m: Nat) {
    is_multiplicative_order_mod(a, n, ord) and a.pow(m).congr_mod(Nat.1, n)
        implies ord.divides(m)
} by {
    if is_multiplicative_order_mod(a, n, ord) and a.pow(m).congr_mod(Nat.1, n) {
        multiplicative_order_positive(a, n, ord)
        division_theorem(m, ord)
        let (q: Nat, r: Nat) satisfy {
            r < ord and m = q * ord + r
        }
        multiplicative_order_pow_congr_one(a, n, ord)
        a.pow(ord).congr_mod(Nat.1, n)
        pow_congr_add_order_multiple(a, n, ord, q, r)
        a.pow(q * ord + r).congr_mod(a.pow(r), n)
        a.pow(m).congr_mod(a.pow(r), n)
        congr_mod_symm(a.pow(m), a.pow(r), n)
        a.pow(r).congr_mod(a.pow(m), n)
        congr_mod_trans(a.pow(r), a.pow(m), Nat.1, n)
        a.pow(r).congr_mod(Nat.1, n)
        if r = Nat.0 {
            m = q * ord
            q * ord = ord * q
            ord * q = m
            ord.divides(m)
        } else {
            Nat.0 < r
            multiplicative_order_minimal(a, n, ord, r)
            ord <= r
            lte_imp_not_lt(ord, r)
            not r < ord
            false
        }
        ord.divides(m)
    }
}

/// If a nonzero exponent gives congruence to one, an explicit multiplicative
/// order is at most that exponent.
theorem multiplicative_order_le_exponent(a: Nat, n: Nat, ord: Nat, m: Nat) {
    is_multiplicative_order_mod(a, n, ord) and m != Nat.0
        and a.pow(m).congr_mod(Nat.1, n)
        implies ord <= m
} by {
    if is_multiplicative_order_mod(a, n, ord) and m != Nat.0
        and a.pow(m).congr_mod(Nat.1, n) {
        multiplicative_order_divides_exponent(a, n, ord, m)
        ord.divides(m)
        divides_lte(ord, m)
        if m = Nat.0 {
            false
        } else {
            ord <= m
        }
    }
}

/// The selected order divides every exponent whose power is congruent to `1`.
theorem multiplicative_order_mod_divides_exponent(a: Nat, n: Nat, m: Nat) {
    n != Nat.0 and a.coprime(n) and a.pow(m).congr_mod(Nat.1, n)
        implies multiplicative_order_mod(a, n).divides(m)
} by {
    if n != Nat.0 and a.coprime(n) and a.pow(m).congr_mod(Nat.1, n) {
        multiplicative_order_mod_is_order(a, n)
        is_multiplicative_order_mod(a, n, multiplicative_order_mod(a, n))
        multiplicative_order_divides_exponent(a, n, multiplicative_order_mod(a, n), m)
    }
}

/// Every multiple of a multiplicative order is an exponent giving congruence to `1`.
theorem multiplicative_order_divides_imp_pow_congr_one(a: Nat, n: Nat, ord: Nat, m: Nat) {
    is_multiplicative_order_mod(a, n, ord) and ord.divides(m)
        implies a.pow(m).congr_mod(Nat.1, n)
} by {
    if is_multiplicative_order_mod(a, n, ord) and ord.divides(m) {
        let q: Nat satisfy { ord * q = m }
        multiplicative_order_pow_congr_one(a, n, ord)
        a.pow(ord).congr_mod(Nat.1, n)
        ord * q = q * ord
        q * ord = m
        pow_mul_order_exponent_congr_one(a, n, ord, q)
        a.pow(q * ord).congr_mod(Nat.1, n)
        a.pow(m).congr_mod(Nat.1, n)
    }
}

/// The selected order turns divisibility of an exponent into congruence to `1`.
theorem multiplicative_order_mod_divides_imp_pow_congr_one(a: Nat, n: Nat, m: Nat) {
    n != Nat.0 and a.coprime(n) and multiplicative_order_mod(a, n).divides(m)
        implies a.pow(m).congr_mod(Nat.1, n)
} by {
    if n != Nat.0 and a.coprime(n) and multiplicative_order_mod(a, n).divides(m) {
        multiplicative_order_mod_is_order(a, n)
        is_multiplicative_order_mod(a, n, multiplicative_order_mod(a, n))
        multiplicative_order_divides_imp_pow_congr_one(a, n, multiplicative_order_mod(a, n), m)
    }
}

/// For an explicit multiplicative order, divisibility of an exponent is
/// equivalent to the corresponding power being congruent to `1`.
theorem multiplicative_order_divides_exponent_iff(a: Nat, n: Nat, ord: Nat, m: Nat) {
    is_multiplicative_order_mod(a, n, ord)
        implies (ord.divides(m) = a.pow(m).congr_mod(Nat.1, n))
} by {
    if is_multiplicative_order_mod(a, n, ord) {
        if ord.divides(m) {
            multiplicative_order_divides_imp_pow_congr_one(a, n, ord, m)
            a.pow(m).congr_mod(Nat.1, n)
        }
        if a.pow(m).congr_mod(Nat.1, n) {
            multiplicative_order_divides_exponent(a, n, ord, m)
            ord.divides(m)
        }
        (ord.divides(m) = a.pow(m).congr_mod(Nat.1, n)) = true
    }
}

/// For the selected multiplicative order, divisibility of an exponent is
/// equivalent to the corresponding power being congruent to `1`.
theorem multiplicative_order_mod_divides_exponent_iff(a: Nat, n: Nat, m: Nat) {
    n != Nat.0 and a.coprime(n)
        implies (multiplicative_order_mod(a, n).divides(m) =
            a.pow(m).congr_mod(Nat.1, n))
} by {
    if n != Nat.0 and a.coprime(n) {
        multiplicative_order_mod_is_order(a, n)
        is_multiplicative_order_mod(a, n, multiplicative_order_mod(a, n))
        multiplicative_order_divides_exponent_iff(a, n, multiplicative_order_mod(a, n), m)
        multiplicative_order_mod(a, n).divides(m) = a.pow(m).congr_mod(Nat.1, n)
    }
}

/// If two powers are congruent, the order divides the sum of the left
/// exponent and a complement of the right exponent modulo the order.
theorem order_divides_exponent_add_complement_of_pow_congr(
    a: Nat, n: Nat, ord: Nat, i: Nat, j: Nat
) {
    is_multiplicative_order_mod(a, n, ord) and i < ord and j < ord
        and a.pow(i).congr_mod(a.pow(j), n)
        implies ord.divides(i + (ord - j))
} by {
    if is_multiplicative_order_mod(a, n, ord) and i < ord and j < ord
        and a.pow(i).congr_mod(a.pow(j), n) {
        multiplicative_order_pow_congr_one(a, n, ord)
        a.pow(ord).congr_mod(Nat.1, n)
        add_sub(ord, j)
        ord - j + j = ord
        j + (ord - j) = ord
        exp_add(a, i, ord - j)
        a.pow(i + (ord - j)) = a.pow(i) * a.pow(ord - j)
        exp_add(a, j, ord - j)
        a.pow(j + (ord - j)) = a.pow(j) * a.pow(ord - j)
        j + (ord - j) = ord
        a.pow(ord) = a.pow(j) * a.pow(ord - j)
        a.pow(j) * a.pow(ord - j) = a.pow(ord)
        congr_mod_refl(a.pow(ord - j), n)
        a.pow(ord - j).congr_mod(a.pow(ord - j), n)
        congr_mod_mul(a.pow(i), a.pow(ord - j), a.pow(j), a.pow(ord - j), n)
        (a.pow(i) * a.pow(ord - j)).congr_mod(a.pow(j) * a.pow(ord - j), n)
        a.pow(i + (ord - j)).congr_mod(a.pow(ord), n)
        congr_mod_trans(a.pow(i + (ord - j)), a.pow(ord), Nat.1, n)
        a.pow(i + (ord - j)).congr_mod(Nat.1, n)
        multiplicative_order_divides_exponent(a, n, ord, i + (ord - j))
        ord.divides(i + (ord - j))
    }
}

/// Under a power congruence below the order, the left exponent cannot be
/// strictly smaller than the right exponent.
theorem powers_below_order_congr_not_lt(
    a: Nat, n: Nat, ord: Nat, i: Nat, j: Nat
) {
    is_multiplicative_order_mod(a, n, ord)
        and i < ord and j < ord and a.pow(i).congr_mod(a.pow(j), n)
        implies not i < j
} by {
    if is_multiplicative_order_mod(a, n, ord)
        and i < ord and j < ord and a.pow(i).congr_mod(a.pow(j), n) {
        if i < j {
            order_divides_exponent_add_complement_of_pow_congr(a, n, ord, i, j)
            ord.divides(i + (ord - j))
            let d: Nat = ord - j
            add_sub(ord, j)
            ord - j + j = ord
            d + j = ord - j + j
            d + j = ord
            j + d = d + j
            j + d = ord
            lt_add_left(d, i, j)
            d + i < d + j
            i + d < ord
            divides_lte(ord, i + d)
            if i + d = Nat.0 {
                i = Nat.0
                d = Nat.0
                j + d = j
                j = ord
                false
            } else {
                ord <= i + d
                lt_and_lte(i + d, ord, i + d)
                false
            }
        }
    }
}

/// Powers with exponents below a multiplicative order are injective modulo the
/// modulus.
theorem powers_below_order_congr_imp_eq(
    a: Nat, n: Nat, ord: Nat, i: Nat, j: Nat
) {
    is_multiplicative_order_mod(a, n, ord)
        and i < ord and j < ord and a.pow(i).congr_mod(a.pow(j), n)
        implies i = j
} by {
    if is_multiplicative_order_mod(a, n, ord)
        and i < ord and j < ord and a.pow(i).congr_mod(a.pow(j), n) {
        powers_below_order_congr_not_lt(a, n, ord, i, j)
        not i < j
        congr_mod_symm(a.pow(i), a.pow(j), n)
        a.pow(j).congr_mod(a.pow(i), n)
        powers_below_order_congr_not_lt(a, n, ord, j, i)
        not j < i
        trichotomy(i, j)
        i = j
    }
}

/// The selected-order form of injectivity for powers below the order.
theorem powers_below_multiplicative_order_mod_congr_imp_eq(
    a: Nat, n: Nat, i: Nat, j: Nat
) {
    n != Nat.0 and a.coprime(n)
        and i < multiplicative_order_mod(a, n) and j < multiplicative_order_mod(a, n)
        and a.pow(i).congr_mod(a.pow(j), n)
        implies i = j
} by {
    if n != Nat.0 and a.coprime(n)
        and i < multiplicative_order_mod(a, n) and j < multiplicative_order_mod(a, n)
        and a.pow(i).congr_mod(a.pow(j), n) {
        multiplicative_order_mod_is_order(a, n)
        is_multiplicative_order_mod(a, n, multiplicative_order_mod(a, n))
        powers_below_order_congr_imp_eq(a, n, multiplicative_order_mod(a, n), i, j)
        i = j
    }
}

/// The selected multiplicative order divides Euler's totient.
theorem multiplicative_order_mod_divides_totient(a: Nat, n: Nat) {
    n != Nat.0 and a.coprime(n) implies multiplicative_order_mod(a, n).divides(n.totient)
} by {
    if n != Nat.0 and a.coprime(n) {
        euler(n, a)
        a.pow(n.totient).congr_mod(Nat.1, n)
        multiplicative_order_mod_divides_exponent(a, n, n.totient)
    }
}

/// The selected multiplicative order is at most Euler's totient on the
/// positive-coprime domain.
theorem multiplicative_order_mod_le_totient(a: Nat, n: Nat) {
    n != Nat.0 and a.coprime(n) implies multiplicative_order_mod(a, n) <= n.totient
} by {
    if n != Nat.0 and a.coprime(n) {
        multiplicative_order_mod_divides_totient(a, n)
        multiplicative_order_mod(a, n).divides(n.totient)
        totient_positive(n)
        Nat.0 < n.totient
        divides_lte(multiplicative_order_mod(a, n), n.totient)
        if n.totient = Nat.0 {
            Nat.0 < Nat.0
            lt_not_ref(Nat.0)
            false
        } else {
            multiplicative_order_mod(a, n) <= n.totient
        }
    }
}

/// Dividing an order and an exponent by their gcd gives the order of the
/// corresponding power.
theorem multiplicative_order_pow_of_gcd_cofactors(
    a: Nat, n: Nat, ord: Nat, k: Nat, q: Nat, r: Nat
) {
    is_multiplicative_order_mod(a, n, ord)
        and q * ord.gcd(k) = ord and r * ord.gcd(k) = k
        implies is_multiplicative_order_mod(a.pow(k), n, q)
} by {
    if is_multiplicative_order_mod(a, n, ord)
        and q * ord.gcd(k) = ord and r * ord.gcd(k) = k {
        let g = ord.gcd(k)
        multiplicative_order_positive(a, n, ord)
        Nat.0 < ord
        g != Nat.0
        if q = Nat.0 {
            q * g = Nat.0
            ord = Nat.0
            false
        }
        q != Nat.0
        Nat.0 < q
        cofactor(ord, k, q, r)
        q.gcd(r) = Nat.1
        q.coprime(r)

        ord * r = (q * g) * r
        (q * g) * r = q * (g * r)
        q * (g * r) = q * (r * g)
        q * (r * g) = (r * g) * q
        (r * g) * q = k * q
        ord * r = k * q
        ord.divides(k * q)
        multiplicative_order_divides_imp_pow_congr_one(a, n, ord, k * q)
        a.pow(k * q).congr_mod(Nat.1, n)
        exp_mul(a, k, q)
        a.pow(k).pow(q) = a.pow(k * q)
        a.pow(k).pow(q).congr_mod(Nat.1, n)
        multiplicative_order_witness(a.pow(k), n)(q)

        forall(j: Nat) {
            if j < q and multiplicative_order_witness(a.pow(k), n)(j) {
                multiplicative_order_witness(a.pow(k), n)(j) =
                    (Nat.0 < j and a.pow(k).pow(j).congr_mod(Nat.1, n))
                Nat.0 < j
                a.pow(k).pow(j).congr_mod(Nat.1, n)
                exp_mul(a, k, j)
                a.pow(k * j).congr_mod(Nat.1, n)
                multiplicative_order_divides_exponent(a, n, ord, k * j)
                ord.divides(k * j)
                (q * g).divides((r * j) * g)
                divides_cancel_right(g, q, r * j)
                q.divides(r * j)
                coprime_divides_of_divides_mul(q, r, j)
                q.divides(j)
                divides_lte(q, j)
                q <= j
                lte_imp_not_lt(q, j)
                false
            }
        }
        false_below(multiplicative_order_witness(a.pow(k), n), q)
        is_min(multiplicative_order_witness(a.pow(k), n), q)
        is_multiplicative_order_mod(a.pow(k), n, q)
    }
}

/// The selected order of a power is the original selected order divided by
/// the gcd of that order with the exponent.
theorem multiplicative_order_mod_pow(a: Nat, n: Nat, k: Nat) {
    n != Nat.0 and a.coprime(n) implies
        multiplicative_order_mod(a.pow(k), n) =
            multiplicative_order_mod(a, n).div(multiplicative_order_mod(a, n).gcd(k))
} by {
    if n != Nat.0 and a.coprime(n) {
        let ord = multiplicative_order_mod(a, n)
        let g = ord.gcd(k)
        multiplicative_order_mod_is_order(a, n)
        is_multiplicative_order_mod(a, n, ord)
        multiplicative_order_mod_positive(a, n)
        Nat.0 < ord
        gcd_divides_left(ord, k)
        gcd_divides_right(ord, k)
        let q: Nat satisfy { g * q = ord }
        let r: Nat satisfy { g * r = k }
        q * g = ord
        r * g = k
        q * ord.gcd(k) = ord
        r * ord.gcd(k) = k
        g != Nat.0
        multiplicative_order_pow_of_gcd_cofactors(a, n, ord, k, q, r)
        is_multiplicative_order_mod(a.pow(k), n, q)
        coprime_pow_right(a, n, k)
        a.pow(k).coprime(n)
        multiplicative_order_mod_eq_of_is_order(a.pow(k), n, q)
        multiplicative_order_mod(a.pow(k), n) = q
        div_mul(q, g)
        (q * g).div(g) = q
        ord.div(g) = q
        multiplicative_order_mod(a.pow(k), n) = ord.div(g)
        multiplicative_order_mod(a.pow(k), n) =
            multiplicative_order_mod(a, n).div(multiplicative_order_mod(a, n).gcd(k))
    }
}

/// The product of elements with coprime multiplicative orders has order equal
/// to the product of those orders.
theorem multiplicative_order_mul_of_coprime_orders(
    a: Nat, b: Nat, n: Nat, r: Nat, s: Nat
) {
    is_multiplicative_order_mod(a, n, r)
        and is_multiplicative_order_mod(b, n, s) and r.coprime(s)
        implies is_multiplicative_order_mod(a * b, n, r * s)
} by {
    if is_multiplicative_order_mod(a, n, r)
        and is_multiplicative_order_mod(b, n, s) and r.coprime(s) {
        multiplicative_order_positive(a, n, r)
        multiplicative_order_positive(b, n, s)
        Nat.0 < r
        Nat.0 < s
        if r * s = Nat.0 {
            mul_to_zero(r, s)
            r = Nat.0 or s = Nat.0
            if r = Nat.0 {
                Nat.0 < Nat.0
                lt_not_ref(Nat.0)
                false
            } else {
                s = Nat.0
                Nat.0 < Nat.0
                lt_not_ref(Nat.0)
                false
            }
        }
        r * s != Nat.0
        Nat.0 < r * s

        r.divides(r * s)
        s.divides(r * s)
        multiplicative_order_divides_imp_pow_congr_one(a, n, r, r * s)
        multiplicative_order_divides_imp_pow_congr_one(b, n, s, r * s)
        a.pow(r * s).congr_mod(Nat.1, n)
        b.pow(r * s).congr_mod(Nat.1, n)
        pow_distrib_mul(a, b, r * s)
        congr_mod_mul(a.pow(r * s), b.pow(r * s), Nat.1, Nat.1, n)
        (a * b).pow(r * s).congr_mod(Nat.1, n)
        multiplicative_order_witness(a * b, n)(r * s)

        forall(k: Nat) {
            if k < r * s and multiplicative_order_witness(a * b, n)(k) {
                multiplicative_order_witness(a * b, n)(k) =
                    (Nat.0 < k and (a * b).pow(k).congr_mod(Nat.1, n))
                Nat.0 < k
                (a * b).pow(k).congr_mod(Nat.1, n)

                congr_mod_pow((a * b).pow(k), Nat.1, n, s)
                (a * b).pow(k).pow(s).congr_mod(Nat.1.pow(s), n)
                exp_mul(a * b, k, s)
                one_exp(s)
                (a * b).pow(k * s).congr_mod(Nat.1, n)
                s.divides(k * s)
                multiplicative_order_divides_imp_pow_congr_one(b, n, s, k * s)
                b.pow(k * s).congr_mod(Nat.1, n)
                pow_distrib_mul(a, b, k * s)
                congr_mod_refl(a.pow(k * s), n)
                congr_mod_mul(a.pow(k * s), b.pow(k * s), a.pow(k * s), Nat.1, n)
                (a * b).pow(k * s).congr_mod(a.pow(k * s), n)
                congr_mod_symm((a * b).pow(k * s), a.pow(k * s), n)
                a.pow(k * s).congr_mod((a * b).pow(k * s), n)
                congr_mod_trans(a.pow(k * s), (a * b).pow(k * s), Nat.1, n)
                a.pow(k * s).congr_mod(Nat.1, n)
                multiplicative_order_divides_exponent(a, n, r, k * s)
                r.divides(k * s)
                coprime_divides_of_divides_mul(r, s, k)
                r.divides(k)

                congr_mod_pow((a * b).pow(k), Nat.1, n, r)
                (a * b).pow(k).pow(r).congr_mod(Nat.1.pow(r), n)
                exp_mul(a * b, k, r)
                one_exp(r)
                (a * b).pow(k * r).congr_mod(Nat.1, n)
                r.divides(k * r)
                multiplicative_order_divides_imp_pow_congr_one(a, n, r, k * r)
                a.pow(k * r).congr_mod(Nat.1, n)
                pow_distrib_mul(a, b, k * r)
                congr_mod_refl(b.pow(k * r), n)
                congr_mod_mul(a.pow(k * r), b.pow(k * r), Nat.1, b.pow(k * r), n)
                (a * b).pow(k * r).congr_mod(b.pow(k * r), n)
                congr_mod_symm((a * b).pow(k * r), b.pow(k * r), n)
                b.pow(k * r).congr_mod((a * b).pow(k * r), n)
                congr_mod_trans(b.pow(k * r), (a * b).pow(k * r), Nat.1, n)
                b.pow(k * r).congr_mod(Nat.1, n)
                multiplicative_order_divides_exponent(b, n, s, k * r)
                s.divides(k * r)
                coprime_comm(r, s)
                s.coprime(r)
                coprime_divides_of_divides_mul(s, r, k)
                s.divides(k)

                nat_coprime_combine(r, s, k)
                (r * s).divides(k)
                divides_lte(r * s, k)
                r * s <= k
                lte_imp_not_lt(r * s, k)
                false
            }
        }
        false_below(multiplicative_order_witness(a * b, n), r * s)
        is_min(multiplicative_order_witness(a * b, n), r * s)
        is_multiplicative_order_mod(a * b, n, r * s)
    }
}

/// Multiplying two units whose selected orders are coprime multiplies their
/// selected orders.
theorem multiplicative_order_mod_mul_of_coprime_orders(a: Nat, b: Nat, n: Nat) {
    n != Nat.0 and a.coprime(n) and b.coprime(n)
        and multiplicative_order_mod(a, n).coprime(multiplicative_order_mod(b, n))
        implies multiplicative_order_mod(a * b, n) =
            multiplicative_order_mod(a, n) * multiplicative_order_mod(b, n)
} by {
    if n != Nat.0 and a.coprime(n) and b.coprime(n)
        and multiplicative_order_mod(a, n).coprime(multiplicative_order_mod(b, n)) {
        let r = multiplicative_order_mod(a, n)
        let s = multiplicative_order_mod(b, n)
        multiplicative_order_mod_is_order(a, n)
        multiplicative_order_mod_is_order(b, n)
        is_multiplicative_order_mod(a, n, r)
        is_multiplicative_order_mod(b, n, s)
        r.coprime(s)
        multiplicative_order_mul_of_coprime_orders(a, b, n, r, s)
        is_multiplicative_order_mod(a * b, n, r * s)
        coprime_comm(a, n)
        coprime_comm(b, n)
        n.coprime(a)
        n.coprime(b)
        coprime_mul(n, a, b)
        n.coprime(a * b)
        coprime_comm(n, a * b)
        (a * b).coprime(n)
        multiplicative_order_mod_eq_of_is_order(a * b, n, r * s)
        multiplicative_order_mod(a * b, n) = r * s
        multiplicative_order_mod(a * b, n) =
            multiplicative_order_mod(a, n) * multiplicative_order_mod(b, n)
    }
}

/// Every nonzero divisor of a selected order is the selected order of a power.
theorem multiplicative_order_mod_pow_to_divisor(a: Nat, n: Nat, d: Nat) {
    n != Nat.0 and a.coprime(n) and d != Nat.0
        and d.divides(multiplicative_order_mod(a, n))
        implies multiplicative_order_mod(
            a.pow(multiplicative_order_mod(a, n).div(d)), n) = d
} by {
    if n != Nat.0 and a.coprime(n) and d != Nat.0
        and d.divides(multiplicative_order_mod(a, n)) {
        let ord = multiplicative_order_mod(a, n)
        let q: Nat satisfy { d * q = ord }
        multiplicative_order_mod_positive(a, n)
        Nat.0 < ord
        if q = Nat.0 {
            d * q = Nat.0
            ord = Nat.0
            false
        }
        q != Nat.0
        div_mul(q, d)
        (q * d).div(d) = q
        q * d = ord
        ord.div(d) = q

        q.divides(ord)
        q.divides(q)
        divides_gcd(q, ord, q)
        q.divides(ord.gcd(q))
        gcd_divides_right(ord, q)
        ord.gcd(q).divides(q)
        divides_symm(q, ord.gcd(q))
        q = ord.gcd(q)
        ord.gcd(q) = q

        multiplicative_order_mod_pow(a, n, q)
        multiplicative_order_mod(a.pow(q), n) = ord.div(ord.gcd(q))
        multiplicative_order_mod(a.pow(q), n) = ord.div(q)
        div_mul(d, q)
        (d * q).div(q) = d
        ord.div(q) = d
        multiplicative_order_mod(a.pow(q), n) = d
        multiplicative_order_mod(
            a.pow(multiplicative_order_mod(a, n).div(d)), n) = d
    }
}

/// Two units modulo a positive modulus determine a unit whose selected order
/// is the least common multiple of their selected orders.
theorem exists_unit_with_multiplicative_order_mod_lcm(
    a: Nat, b: Nat, n: Nat
) {
    n != Nat.0 and a.coprime(n) and b.coprime(n) implies exists(c: Nat) {
        c.coprime(n) and multiplicative_order_mod(c, n) =
            multiplicative_order_mod(a, n).lcm(multiplicative_order_mod(b, n))
    }
} by {
    if n != Nat.0 and a.coprime(n) and b.coprime(n) {
        let r = multiplicative_order_mod(a, n)
        let s = multiplicative_order_mod(b, n)
        multiplicative_order_mod_positive(a, n)
        multiplicative_order_mod_positive(b, n)
        Nat.0 < r
        Nat.0 < s
        r != Nat.0
        s != Nat.0
        coprime_factors_mul_lcm(r, s)
        let (left: Nat, right: Nat) satisfy {
            left != Nat.0 and right != Nat.0
                and left.divides(r) and right.divides(s)
                and left.coprime(right) and left * right = r.lcm(s)
        }
        let x = a.pow(r.div(left))
        let y = b.pow(s.div(right))
        multiplicative_order_mod_pow_to_divisor(a, n, left)
        multiplicative_order_mod_pow_to_divisor(b, n, right)
        multiplicative_order_mod(a.pow(r.div(left)), n) = left
        multiplicative_order_mod(b.pow(s.div(right)), n) = right
        multiplicative_order_mod(x, n) = left
        multiplicative_order_mod(y, n) = right
        coprime_pow_right(a, n, r.div(left))
        coprime_pow_right(b, n, s.div(right))
        x.coprime(n)
        y.coprime(n)
        multiplicative_order_mod(x, n).coprime(multiplicative_order_mod(y, n))
        multiplicative_order_mod_mul_of_coprime_orders(x, y, n)
        multiplicative_order_mod(x * y, n) =
            multiplicative_order_mod(x, n) * multiplicative_order_mod(y, n)
        multiplicative_order_mod(x * y, n) = left * right
        multiplicative_order_mod(x * y, n) = r.lcm(s)

        coprime_comm(x, n)
        coprime_comm(y, n)
        n.coprime(x)
        n.coprime(y)
        coprime_mul(n, x, y)
        n.coprime(x * y)
        coprime_comm(n, x * y)
        (x * y).coprime(n)
        exists(c: Nat) {
            c.coprime(n) and multiplicative_order_mod(c, n) =
                multiplicative_order_mod(a, n).lcm(
                    multiplicative_order_mod(b, n))
        }
    }
}
