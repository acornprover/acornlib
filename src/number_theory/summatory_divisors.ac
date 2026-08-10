/// Summatory divisor functions.
///
/// This module collects the summatory (partial-sum) identities around the
/// divisor-sum functions `sigma` and `tau`:
///
///   (a) the definitional identity `sigma(n) = sum_{d | n} d`;
///   (b) the closed forms `sum_{k=1}^{n} k = n(n+1)/2` and
///       `sum_{k=1}^{n} k^2 = n(n+1)(2n+1)/6` (restated from
///       `sequence_identities.ac`);
///   (c) the classical summatory identity
///       `sum_{k=1}^{n} sigma(k) = sum_{d=1}^{n} d * floor(n / d)`,
///       stated in full and verified at `n = 0, 1, 2, 3, 4` (the general
///       proof needs the interchange of the two summations, the same piece
///       recorded as missing at the end of `mobius_inversion.ac`);
///   (d) the cofactor symmetry `sum_{d | n} tau(d) = sum_{d | n} tau(n / d)`,
///       the natural-valued form of the identity
///       `sum_{d | n} tau(d) = sum_{d | n} d * tau(n / d) * (1 / d)`;
///   (e) the summatory totient `Phi(n) = sum_{k=1}^{n} phi(k)`, with the
///       asymptotic `Phi(n) ~ 3 n^2 / pi^2` recorded as a comment (it needs
///       real analysis with the constant `pi`, which the library does not
///       provide yet).
from nat import Nat, mul_zero_left, mul_one_right, mul_one_left, add_assoc
from nat import div_mul, div_of_decomp
from rat import Rat
from list import List, map, sum
from data.nat.nat_range_sum import range_sum, range_sum_zero, range_sum_one, range_sum_suc
from number_theory.divisor_sum import divisor_list, divisor_sum_fn, divisor_sum_fn_apply,
    divisor_sum_fn_at_zero, divisor_list_zero, nat_sigma, nat_tau, nat_sigma_zero,
    nat_sigma_one, nat_sigma_prime, divisor_sum_fn_nat_identity_arithmetic_fn_eq_sigma,
    sum_map_nat_identity_arithmetic_fn_eq_sum
from number_theory.arithmetic_functions import nat_identity_arithmetic_fn
from number_theory.dirichlet import divisor_quotient
from number_theory.totient_sums import divisor_quotient_reindex
from number_theory.totient import nat_totient, nat_totient_zero, nat_totient_one
from number_theory.sequence_identities import seq_triangular_sum, seq_sum_of_squares
from number_theory.sum_of_powers import power_sum, closed_first, closed_squares
from number_theory.falling_product import nat_two_prime
from number_theory.carmichael import three_is_prime
from number_theory.perfect_numbers import nat_sigma_four
numerals Nat
numerals Rat

// ---------------------------------------------------------------------------
// (a) sigma as the divisor sum of the identity function.
//
// The definition `nat_sigma(n) = sum(divisor_list(n))` in `divisor_sum.ac`
// already identifies `sigma` with the sum of the positive divisors of `n`;
// the two theorems below restate it in the divisor-sum notation of the
// library, `sum_{d | n} d`.
// ---------------------------------------------------------------------------

/// `sigma(n) = sum_{d | n} d`: the divisor sum of the identity arithmetic
/// function at `n` is `sigma(n)`.
theorem nat_sigma_eq_divisor_sum(n: Nat) {
    nat_sigma(n) = divisor_sum_fn(nat_identity_arithmetic_fn)(n)
} by {
    divisor_sum_fn_nat_identity_arithmetic_fn_eq_sigma(n)
    divisor_sum_fn(nat_identity_arithmetic_fn)(n) = nat_sigma(n)
}

/// `sigma(n) = sum_{d | n} d` in the divisor-list form:
/// `nat_sigma(n) = sum(map(divisor_list(n), identity))`.
theorem nat_sigma_eq_divisor_list_sum(n: Nat) {
    nat_sigma(n) = sum(map(divisor_list(n), nat_identity_arithmetic_fn))
} by {
    sum_map_nat_identity_arithmetic_fn_eq_sum(divisor_list(n))
    sum(map(divisor_list(n), nat_identity_arithmetic_fn)) = sum(divisor_list(n))
    nat_sigma(n) = sum(divisor_list(n))
    nat_sigma(n) = sum(map(divisor_list(n), nat_identity_arithmetic_fn))
}

// ---------------------------------------------------------------------------
// (b) The classical power-sum closed forms.
//
// Both identities are proved in `number_theory/sum_of_powers.ac` (via
// Faulhaber-style recurrences) and restated in `sequence_identities.ac`; the
// theorems below restate them here as the summatory building blocks used by
// the summatory divisor functions.
// ---------------------------------------------------------------------------

/// The triangular numbers: `sum_{k=1}^{n} k = n(n+1)/2`.
///
/// Restates `seq_triangular_sum` from `sequence_identities.ac`, where the sum
/// is `power_sum(Nat.1, n) = 0^1 + 1^1 + ... + n^1` (the k = 0 term is zero)
/// and the closed form is `closed_first(n) = n(n+1)/2` in the rationals.
theorem summatory_first_sum(n: Nat) {
    power_sum(Nat.1, n) = closed_first(n)
} by {
    seq_triangular_sum(n)
    power_sum(Nat.1, n) = closed_first(n)
}

/// The sum of the first `n` squares: `sum_{k=1}^{n} k^2 = n(n+1)(2n+1)/6`.
///
/// Restates `seq_sum_of_squares` from `sequence_identities.ac`, with the
/// closed form `closed_squares(n) = n(n+1)(2n+1)/6`.
theorem summatory_squares_sum(n: Nat) {
    power_sum(Nat.2, n) = closed_squares(n)
} by {
    seq_sum_of_squares(n)
    power_sum(Nat.2, n) = closed_squares(n)
}

// ---------------------------------------------------------------------------
// (c) The summatory sigma and the floor-weighted identity.
//
// The classical identity
//
//     sum_{k=1}^{n} sigma(k) = sum_{d=1}^{n} d * floor(n / d)
//
// follows by expanding sigma(k) = sum_{d | k} d and interchanging the two
// summations: the number of multiples of d among 1, ..., n is floor(n / d).
// The interchange is a permutation argument on the pairs (k, d) with d | k,
// the same piece recorded as missing at the end of `mobius_inversion.ac` and
// `totient_sums.ac`; the identity is therefore stated in full as a comment and
// verified below at n = 0, 1, 2, 3, 4.
// ---------------------------------------------------------------------------

/// The summatory divisor-sum function: `S(n) = sum_{k=1}^{n} sigma(k)`.
///
/// As a range sum this is `range_sum(nat_sigma, n.suc) = sigma(0) + ... +
/// sigma(n)`, and `sigma(0) = 0`, so the k = 0 term contributes nothing.
define summatory_sigma(n: Nat) -> Nat {
    range_sum(nat_sigma, n.suc)
}

/// The term `d * floor(n / d)` of the floor-weighted summatory sum.
define floor_weight_term(n: Nat, d: Nat) -> Nat {
    d * (n.div(d))
}

/// The floor-weighted summatory sum: `T(n) = sum_{d=1}^{n} d * floor(n / d)`.
///
/// The d = 0 term is `0 * floor(n / 0) = 0`, so summing over `[0, n]` via
/// `range_sum` matches the sum over `[1, n]`.
define floor_weighted_sigma_sum(n: Nat) -> Nat {
    range_sum(function(d: Nat) { floor_weight_term(n, d) }, n.suc)
}

// ---------------------------------------------------------------------------
// (c.1) Small quotient facts.
// ---------------------------------------------------------------------------

/// Dividing by one is the identity: `n div 1 = n`.
theorem nat_div_one(n: Nat) {
    n.div(Nat.1) = n
} by {
    div_mul(n, Nat.1)
    (n * Nat.1).div(Nat.1) = n
    mul_one_right(n)
    n * Nat.1 = n
    n.div(Nat.1) = n
}

/// Dividing a nonzero number by itself gives one: `n div n = 1`.
theorem nat_div_self_one(n: Nat) {
    n != Nat.0 implies n.div(n) = Nat.1
} by {
    if n != Nat.0 {
        div_mul(Nat.1, n)
        (Nat.1 * n).div(n) = Nat.1
        mul_one_left(n)
        Nat.1 * n = n
        n.div(n) = Nat.1
    }
}

/// `3 div 2 = 1`, since `3 = 1 * 2 + 1`.
theorem nat_div_three_two {
    Nat.3.div(Nat.2) = Nat.1
} by {
    div_of_decomp(Nat.1, Nat.1, Nat.2)
    Nat.1 < Nat.2
    (Nat.1 * Nat.2 + Nat.1).div(Nat.2) = Nat.1
    Nat.1 * Nat.2 + Nat.1 = Nat.3
    Nat.3.div(Nat.2) = Nat.1
}

/// `4 div 3 = 1`, since `4 = 1 * 3 + 1`.
theorem nat_div_four_three {
    Nat.4.div(Nat.3) = Nat.1
} by {
    div_of_decomp(Nat.1, Nat.1, Nat.3)
    Nat.1 < Nat.3
    (Nat.1 * Nat.3 + Nat.1).div(Nat.3) = Nat.1
    Nat.1 * Nat.3 + Nat.1 = Nat.4
    Nat.4.div(Nat.3) = Nat.1
}

/// `4 div 2 = 2`, since `4 = 2 * 2`.
theorem nat_div_four_two {
    Nat.4.div(Nat.2) = Nat.2
} by {
    div_mul(Nat.2, Nat.2)
    (Nat.2 * Nat.2).div(Nat.2) = Nat.2
    Nat.2 * Nat.2 = Nat.4
    Nat.4.div(Nat.2) = Nat.2
}

// ---------------------------------------------------------------------------
// (c.1b) Small decimal additions.
//
// The automation computes decimal addition only for a few digit orders, so the
// additions with carries used by the small cases below are recorded as lemmas.
// ---------------------------------------------------------------------------

/// `3 + 2 = 5`.
theorem nat_add_3_2 {
    Nat.3 + Nat.2 = Nat.5
}

/// `8 + 7 = 15`, decomposed as `(8 + 2) + 5 = 10 + 5`.
theorem nat_add_8_7 {
    Nat.8 + Nat.7 = Nat.15
} by {
    Nat.2 + Nat.5 = Nat.7
    Nat.8 + Nat.7 = Nat.8 + (Nat.2 + Nat.5)
    add_assoc(Nat.8, Nat.2, Nat.5)
    Nat.8 + (Nat.2 + Nat.5) = (Nat.8 + Nat.2) + Nat.5
    Nat.8 + Nat.2 = Nat.10
    (Nat.8 + Nat.2) + Nat.5 = Nat.10 + Nat.5
    Nat.10 + Nat.5 = Nat.15
    Nat.8 + Nat.7 = Nat.15
}

/// `8 + 3 = 11`, decomposed as `(8 + 2) + 1 = 10 + 1`.
theorem nat_add_8_3 {
    Nat.8 + Nat.3 = Nat.11
} by {
    Nat.2 + Nat.1 = Nat.3
    Nat.8 + Nat.3 = Nat.8 + (Nat.2 + Nat.1)
    add_assoc(Nat.8, Nat.2, Nat.1)
    Nat.8 + (Nat.2 + Nat.1) = (Nat.8 + Nat.2) + Nat.1
    Nat.8 + Nat.2 = Nat.10
    (Nat.8 + Nat.2) + Nat.1 = Nat.10 + Nat.1
    Nat.10 + Nat.1 = Nat.11
    Nat.8 + Nat.3 = Nat.11
}

/// `11 + 4 = 15`.
theorem nat_add_11_4 {
    Nat.11 + Nat.4 = Nat.15
}

// ---------------------------------------------------------------------------
// (c.2) Small values of sigma.
// ---------------------------------------------------------------------------

/// `sigma(2) = 3`, since two is prime.
theorem nat_sigma_two {
    nat_sigma(Nat.2) = Nat.3
} by {
    nat_two_prime
    nat_sigma_prime(Nat.2)
    nat_sigma(Nat.2) = Nat.2 + Nat.1
    Nat.2 + Nat.1 = Nat.3
}

/// `sigma(3) = 4`, since three is prime.
theorem nat_sigma_three {
    nat_sigma(Nat.3) = Nat.4
} by {
    three_is_prime
    nat_sigma_prime(Nat.3)
    nat_sigma(Nat.3) = Nat.3 + Nat.1
    Nat.3 + Nat.1 = Nat.4
}

/// `sigma(4) = 7`, since the divisors of four are `1, 2, 4`.
///
/// Restates `nat_sigma_four` from `perfect_numbers.ac`.
theorem nat_sigma_four_value {
    nat_sigma(Nat.4) = Nat.7
} by {
    nat_sigma_four
}

// ---------------------------------------------------------------------------
// (c.3) The small cases of the summatory identity.
// ---------------------------------------------------------------------------

/// `S(0) = 0`, the empty summatory sum.
theorem summatory_sigma_zero_value {
    summatory_sigma(Nat.0) = Nat.0
} by {
    summatory_sigma(Nat.0) = range_sum(nat_sigma, Nat.1)
    range_sum_one(nat_sigma)
    range_sum(nat_sigma, Nat.1) = nat_sigma(Nat.0)
    nat_sigma_zero
    nat_sigma(Nat.0) = Nat.0
    summatory_sigma(Nat.0) = Nat.0
}

/// `S(1) = 1`, since `sigma(1) = 1`.
theorem summatory_sigma_one_value {
    summatory_sigma(Nat.1) = Nat.1
} by {
    summatory_sigma(Nat.1) = range_sum(nat_sigma, Nat.2)
    range_sum_suc(nat_sigma, Nat.1)
    range_sum(nat_sigma, Nat.2) = range_sum(nat_sigma, Nat.1) + nat_sigma(Nat.1)
    range_sum_suc(nat_sigma, Nat.0)
    range_sum(nat_sigma, Nat.1) = range_sum(nat_sigma, Nat.0) + nat_sigma(Nat.0)
    range_sum_zero(nat_sigma)
    range_sum(nat_sigma, Nat.0) = Nat.0
    nat_sigma_zero
    nat_sigma(Nat.0) = Nat.0
    nat_sigma_one
    nat_sigma(Nat.1) = Nat.1
    range_sum(nat_sigma, Nat.1) = Nat.0
    range_sum(nat_sigma, Nat.2) = Nat.1
    summatory_sigma(Nat.1) = Nat.1
}

/// `S(2) = 4`, since `sigma(1) + sigma(2) = 1 + 3`.
theorem summatory_sigma_two_value {
    summatory_sigma(Nat.2) = Nat.4
} by {
    summatory_sigma(Nat.2) = range_sum(nat_sigma, Nat.3)
    range_sum_suc(nat_sigma, Nat.2)
    range_sum(nat_sigma, Nat.3) = range_sum(nat_sigma, Nat.2) + nat_sigma(Nat.2)
    range_sum_suc(nat_sigma, Nat.1)
    range_sum(nat_sigma, Nat.2) = range_sum(nat_sigma, Nat.1) + nat_sigma(Nat.1)
    range_sum_suc(nat_sigma, Nat.0)
    range_sum(nat_sigma, Nat.1) = range_sum(nat_sigma, Nat.0) + nat_sigma(Nat.0)
    range_sum_zero(nat_sigma)
    range_sum(nat_sigma, Nat.0) = Nat.0
    nat_sigma_zero
    nat_sigma(Nat.0) = Nat.0
    nat_sigma_one
    nat_sigma(Nat.1) = Nat.1
    nat_sigma_two
    nat_sigma(Nat.2) = Nat.3
    range_sum(nat_sigma, Nat.1) = Nat.0
    range_sum(nat_sigma, Nat.2) = Nat.1
    range_sum(nat_sigma, Nat.3) = Nat.4
    summatory_sigma(Nat.2) = Nat.4
}

/// `S(3) = 8`, since `S(2) + sigma(3) = 4 + 4`.
theorem summatory_sigma_three_value {
    summatory_sigma(Nat.3) = Nat.8
} by {
    summatory_sigma(Nat.3) = range_sum(nat_sigma, Nat.4)
    range_sum_suc(nat_sigma, Nat.3)
    range_sum(nat_sigma, Nat.4) = range_sum(nat_sigma, Nat.3) + nat_sigma(Nat.3)
    summatory_sigma_two_value
    summatory_sigma(Nat.2) = Nat.4
    summatory_sigma(Nat.2) = range_sum(nat_sigma, Nat.3)
    range_sum(nat_sigma, Nat.3) = Nat.4
    nat_sigma_three
    nat_sigma(Nat.3) = Nat.4
    Nat.4 + Nat.4 = Nat.8
    range_sum(nat_sigma, Nat.4) = Nat.8
    summatory_sigma(Nat.3) = Nat.8
}

/// `S(4) = 15`, since `S(3) + sigma(4) = 8 + 7`.
theorem summatory_sigma_four_value {
    summatory_sigma(Nat.4) = Nat.15
} by {
    summatory_sigma(Nat.4) = range_sum(nat_sigma, Nat.5)
    range_sum_suc(nat_sigma, Nat.4)
    range_sum(nat_sigma, Nat.5) = range_sum(nat_sigma, Nat.4) + nat_sigma(Nat.4)
    summatory_sigma_three_value
    summatory_sigma(Nat.3) = Nat.8
    summatory_sigma(Nat.3) = range_sum(nat_sigma, Nat.4)
    range_sum(nat_sigma, Nat.4) = Nat.8
    nat_sigma_four_value
    nat_sigma(Nat.4) = Nat.7
    nat_add_8_7
    Nat.8 + Nat.7 = Nat.15
    range_sum(nat_sigma, Nat.5) = Nat.15
    summatory_sigma(Nat.4) = Nat.15
}

/// `T(0) = 0`, the empty floor-weighted sum.
theorem floor_weighted_sigma_sum_zero_value {
    floor_weighted_sigma_sum(Nat.0) = Nat.0
} by {
    floor_weighted_sigma_sum(Nat.0) =
        range_sum(function(d: Nat) { floor_weight_term(Nat.0, d) }, Nat.1)
    range_sum_one(function(d: Nat) { floor_weight_term(Nat.0, d) })
    range_sum(function(d: Nat) { floor_weight_term(Nat.0, d) }, Nat.1) =
        function(d: Nat) { floor_weight_term(Nat.0, d) }(Nat.0)
    function(d: Nat) { floor_weight_term(Nat.0, d) }(Nat.0) =
        floor_weight_term(Nat.0, Nat.0)
    floor_weight_term(Nat.0, Nat.0) = Nat.0 * (Nat.0.div(Nat.0))
    mul_zero_left(Nat.0.div(Nat.0))
    Nat.0 * (Nat.0.div(Nat.0)) = Nat.0
    function(d: Nat) { floor_weight_term(Nat.0, d) }(Nat.0) = Nat.0
    floor_weighted_sigma_sum(Nat.0) = Nat.0
}

/// `T(1) = 1`, since the only term is `1 * floor(1/1) = 1`.
theorem floor_weighted_sigma_sum_one_value {
    floor_weighted_sigma_sum(Nat.1) = Nat.1
} by {
    floor_weighted_sigma_sum(Nat.1) =
        range_sum(function(d: Nat) { floor_weight_term(Nat.1, d) }, Nat.2)
    range_sum_suc(function(d: Nat) { floor_weight_term(Nat.1, d) }, Nat.1)
    range_sum(function(d: Nat) { floor_weight_term(Nat.1, d) }, Nat.2) =
        range_sum(function(d: Nat) { floor_weight_term(Nat.1, d) }, Nat.1) +
            function(d: Nat) { floor_weight_term(Nat.1, d) }(Nat.1)
    range_sum_suc(function(d: Nat) { floor_weight_term(Nat.1, d) }, Nat.0)
    range_sum(function(d: Nat) { floor_weight_term(Nat.1, d) }, Nat.1) =
        range_sum(function(d: Nat) { floor_weight_term(Nat.1, d) }, Nat.0) +
            function(d: Nat) { floor_weight_term(Nat.1, d) }(Nat.0)
    range_sum_zero(function(d: Nat) { floor_weight_term(Nat.1, d) })
    range_sum(function(d: Nat) { floor_weight_term(Nat.1, d) }, Nat.0) = Nat.0
    function(d: Nat) { floor_weight_term(Nat.1, d) }(Nat.0) =
        floor_weight_term(Nat.1, Nat.0)
    floor_weight_term(Nat.1, Nat.0) = Nat.0 * (Nat.1.div(Nat.0))
    mul_zero_left(Nat.1.div(Nat.0))
    Nat.0 * (Nat.1.div(Nat.0)) = Nat.0
    function(d: Nat) { floor_weight_term(Nat.1, d) }(Nat.0) = Nat.0
    function(d: Nat) { floor_weight_term(Nat.1, d) }(Nat.1) =
        floor_weight_term(Nat.1, Nat.1)
    floor_weight_term(Nat.1, Nat.1) = Nat.1 * (Nat.1.div(Nat.1))
    nat_div_self_one(Nat.1)
    Nat.1.div(Nat.1) = Nat.1
    mul_one_left(Nat.1)
    Nat.1 * Nat.1 = Nat.1
    function(d: Nat) { floor_weight_term(Nat.1, d) }(Nat.1) = Nat.1
    range_sum(function(d: Nat) { floor_weight_term(Nat.1, d) }, Nat.1) = Nat.0
    range_sum(function(d: Nat) { floor_weight_term(Nat.1, d) }, Nat.2) = Nat.1
    floor_weighted_sigma_sum(Nat.1) = Nat.1
}

/// `T(2) = 4`, since `1 * floor(2/1) + 2 * floor(2/2) = 2 + 2`.
theorem floor_weighted_sigma_sum_two_value {
    floor_weighted_sigma_sum(Nat.2) = Nat.4
} by {
    floor_weighted_sigma_sum(Nat.2) =
        range_sum(function(d: Nat) { floor_weight_term(Nat.2, d) }, Nat.3)
    range_sum_suc(function(d: Nat) { floor_weight_term(Nat.2, d) }, Nat.2)
    range_sum(function(d: Nat) { floor_weight_term(Nat.2, d) }, Nat.3) =
        range_sum(function(d: Nat) { floor_weight_term(Nat.2, d) }, Nat.2) +
            function(d: Nat) { floor_weight_term(Nat.2, d) }(Nat.2)
    range_sum_suc(function(d: Nat) { floor_weight_term(Nat.2, d) }, Nat.1)
    range_sum(function(d: Nat) { floor_weight_term(Nat.2, d) }, Nat.2) =
        range_sum(function(d: Nat) { floor_weight_term(Nat.2, d) }, Nat.1) +
            function(d: Nat) { floor_weight_term(Nat.2, d) }(Nat.1)
    range_sum_suc(function(d: Nat) { floor_weight_term(Nat.2, d) }, Nat.0)
    range_sum(function(d: Nat) { floor_weight_term(Nat.2, d) }, Nat.1) =
        range_sum(function(d: Nat) { floor_weight_term(Nat.2, d) }, Nat.0) +
            function(d: Nat) { floor_weight_term(Nat.2, d) }(Nat.0)
    range_sum_zero(function(d: Nat) { floor_weight_term(Nat.2, d) })
    range_sum(function(d: Nat) { floor_weight_term(Nat.2, d) }, Nat.0) = Nat.0
    function(d: Nat) { floor_weight_term(Nat.2, d) }(Nat.0) =
        floor_weight_term(Nat.2, Nat.0)
    floor_weight_term(Nat.2, Nat.0) = Nat.0 * (Nat.2.div(Nat.0))
    mul_zero_left(Nat.2.div(Nat.0))
    Nat.0 * (Nat.2.div(Nat.0)) = Nat.0
    function(d: Nat) { floor_weight_term(Nat.2, d) }(Nat.0) = Nat.0
    function(d: Nat) { floor_weight_term(Nat.2, d) }(Nat.1) =
        floor_weight_term(Nat.2, Nat.1)
    floor_weight_term(Nat.2, Nat.1) = Nat.1 * (Nat.2.div(Nat.1))
    nat_div_one(Nat.2)
    Nat.2.div(Nat.1) = Nat.2
    mul_one_left(Nat.2)
    Nat.1 * Nat.2 = Nat.2
    function(d: Nat) { floor_weight_term(Nat.2, d) }(Nat.1) = Nat.2
    function(d: Nat) { floor_weight_term(Nat.2, d) }(Nat.2) =
        floor_weight_term(Nat.2, Nat.2)
    floor_weight_term(Nat.2, Nat.2) = Nat.2 * (Nat.2.div(Nat.2))
    nat_div_self_one(Nat.2)
    Nat.2.div(Nat.2) = Nat.1
    Nat.2 * Nat.1 = Nat.2
    function(d: Nat) { floor_weight_term(Nat.2, d) }(Nat.2) = Nat.2
    range_sum(function(d: Nat) { floor_weight_term(Nat.2, d) }, Nat.1) = Nat.0
    range_sum(function(d: Nat) { floor_weight_term(Nat.2, d) }, Nat.2) = Nat.2
    range_sum(function(d: Nat) { floor_weight_term(Nat.2, d) }, Nat.3) = Nat.4
    floor_weighted_sigma_sum(Nat.2) = Nat.4
}

/// `T(3) = 8`, since `1*3 + 2*1 + 3*1 = 3 + 2 + 3`.
theorem floor_weighted_sigma_sum_three_value {
    floor_weighted_sigma_sum(Nat.3) = Nat.8
} by {
    floor_weighted_sigma_sum(Nat.3) =
        range_sum(function(d: Nat) { floor_weight_term(Nat.3, d) }, Nat.4)
    range_sum_suc(function(d: Nat) { floor_weight_term(Nat.3, d) }, Nat.3)
    range_sum(function(d: Nat) { floor_weight_term(Nat.3, d) }, Nat.4) =
        range_sum(function(d: Nat) { floor_weight_term(Nat.3, d) }, Nat.3) +
            function(d: Nat) { floor_weight_term(Nat.3, d) }(Nat.3)
    range_sum_suc(function(d: Nat) { floor_weight_term(Nat.3, d) }, Nat.2)
    range_sum(function(d: Nat) { floor_weight_term(Nat.3, d) }, Nat.3) =
        range_sum(function(d: Nat) { floor_weight_term(Nat.3, d) }, Nat.2) +
            function(d: Nat) { floor_weight_term(Nat.3, d) }(Nat.2)
    range_sum_suc(function(d: Nat) { floor_weight_term(Nat.3, d) }, Nat.1)
    range_sum(function(d: Nat) { floor_weight_term(Nat.3, d) }, Nat.2) =
        range_sum(function(d: Nat) { floor_weight_term(Nat.3, d) }, Nat.1) +
            function(d: Nat) { floor_weight_term(Nat.3, d) }(Nat.1)
    range_sum_suc(function(d: Nat) { floor_weight_term(Nat.3, d) }, Nat.0)
    range_sum(function(d: Nat) { floor_weight_term(Nat.3, d) }, Nat.1) =
        range_sum(function(d: Nat) { floor_weight_term(Nat.3, d) }, Nat.0) +
            function(d: Nat) { floor_weight_term(Nat.3, d) }(Nat.0)
    range_sum_zero(function(d: Nat) { floor_weight_term(Nat.3, d) })
    range_sum(function(d: Nat) { floor_weight_term(Nat.3, d) }, Nat.0) = Nat.0
    function(d: Nat) { floor_weight_term(Nat.3, d) }(Nat.0) =
        floor_weight_term(Nat.3, Nat.0)
    floor_weight_term(Nat.3, Nat.0) = Nat.0 * (Nat.3.div(Nat.0))
    mul_zero_left(Nat.3.div(Nat.0))
    Nat.0 * (Nat.3.div(Nat.0)) = Nat.0
    function(d: Nat) { floor_weight_term(Nat.3, d) }(Nat.0) = Nat.0
    function(d: Nat) { floor_weight_term(Nat.3, d) }(Nat.1) =
        floor_weight_term(Nat.3, Nat.1)
    floor_weight_term(Nat.3, Nat.1) = Nat.1 * (Nat.3.div(Nat.1))
    nat_div_one(Nat.3)
    Nat.3.div(Nat.1) = Nat.3
    mul_one_left(Nat.3)
    Nat.1 * Nat.3 = Nat.3
    function(d: Nat) { floor_weight_term(Nat.3, d) }(Nat.1) = Nat.3
    function(d: Nat) { floor_weight_term(Nat.3, d) }(Nat.2) =
        floor_weight_term(Nat.3, Nat.2)
    floor_weight_term(Nat.3, Nat.2) = Nat.2 * (Nat.3.div(Nat.2))
    nat_div_three_two
    Nat.3.div(Nat.2) = Nat.1
    Nat.2 * Nat.1 = Nat.2
    function(d: Nat) { floor_weight_term(Nat.3, d) }(Nat.2) = Nat.2
    function(d: Nat) { floor_weight_term(Nat.3, d) }(Nat.3) =
        floor_weight_term(Nat.3, Nat.3)
    floor_weight_term(Nat.3, Nat.3) = Nat.3 * (Nat.3.div(Nat.3))
    nat_div_self_one(Nat.3)
    Nat.3.div(Nat.3) = Nat.1
    Nat.3 * Nat.1 = Nat.3
    function(d: Nat) { floor_weight_term(Nat.3, d) }(Nat.3) = Nat.3
    range_sum(function(d: Nat) { floor_weight_term(Nat.3, d) }, Nat.1) = Nat.0
    range_sum(function(d: Nat) { floor_weight_term(Nat.3, d) }, Nat.2) = Nat.3
    range_sum(function(d: Nat) { floor_weight_term(Nat.3, d) }, Nat.2) +
        function(d: Nat) { floor_weight_term(Nat.3, d) }(Nat.2) = Nat.3 + Nat.2
    nat_add_3_2
    Nat.3 + Nat.2 = Nat.5
    range_sum(function(d: Nat) { floor_weight_term(Nat.3, d) }, Nat.3) = Nat.5
    range_sum(function(d: Nat) { floor_weight_term(Nat.3, d) }, Nat.4) = Nat.8
    floor_weighted_sigma_sum(Nat.3) = Nat.8
}

/// `T(4) = 15`, since `1*4 + 2*2 + 3*1 + 4*1 = 4 + 4 + 3 + 4`.
theorem floor_weighted_sigma_sum_four_value {
    floor_weighted_sigma_sum(Nat.4) = Nat.15
} by {
    floor_weighted_sigma_sum(Nat.4) =
        range_sum(function(d: Nat) { floor_weight_term(Nat.4, d) }, Nat.5)
    range_sum_suc(function(d: Nat) { floor_weight_term(Nat.4, d) }, Nat.4)
    range_sum(function(d: Nat) { floor_weight_term(Nat.4, d) }, Nat.5) =
        range_sum(function(d: Nat) { floor_weight_term(Nat.4, d) }, Nat.4) +
            function(d: Nat) { floor_weight_term(Nat.4, d) }(Nat.4)
    range_sum_suc(function(d: Nat) { floor_weight_term(Nat.4, d) }, Nat.3)
    range_sum(function(d: Nat) { floor_weight_term(Nat.4, d) }, Nat.4) =
        range_sum(function(d: Nat) { floor_weight_term(Nat.4, d) }, Nat.3) +
            function(d: Nat) { floor_weight_term(Nat.4, d) }(Nat.3)
    range_sum_suc(function(d: Nat) { floor_weight_term(Nat.4, d) }, Nat.2)
    range_sum(function(d: Nat) { floor_weight_term(Nat.4, d) }, Nat.3) =
        range_sum(function(d: Nat) { floor_weight_term(Nat.4, d) }, Nat.2) +
            function(d: Nat) { floor_weight_term(Nat.4, d) }(Nat.2)
    range_sum_suc(function(d: Nat) { floor_weight_term(Nat.4, d) }, Nat.1)
    range_sum(function(d: Nat) { floor_weight_term(Nat.4, d) }, Nat.2) =
        range_sum(function(d: Nat) { floor_weight_term(Nat.4, d) }, Nat.1) +
            function(d: Nat) { floor_weight_term(Nat.4, d) }(Nat.1)
    range_sum_suc(function(d: Nat) { floor_weight_term(Nat.4, d) }, Nat.0)
    range_sum(function(d: Nat) { floor_weight_term(Nat.4, d) }, Nat.1) =
        range_sum(function(d: Nat) { floor_weight_term(Nat.4, d) }, Nat.0) +
            function(d: Nat) { floor_weight_term(Nat.4, d) }(Nat.0)
    range_sum_zero(function(d: Nat) { floor_weight_term(Nat.4, d) })
    range_sum(function(d: Nat) { floor_weight_term(Nat.4, d) }, Nat.0) = Nat.0
    function(d: Nat) { floor_weight_term(Nat.4, d) }(Nat.0) =
        floor_weight_term(Nat.4, Nat.0)
    floor_weight_term(Nat.4, Nat.0) = Nat.0 * (Nat.4.div(Nat.0))
    mul_zero_left(Nat.4.div(Nat.0))
    Nat.0 * (Nat.4.div(Nat.0)) = Nat.0
    function(d: Nat) { floor_weight_term(Nat.4, d) }(Nat.0) = Nat.0
    function(d: Nat) { floor_weight_term(Nat.4, d) }(Nat.1) =
        floor_weight_term(Nat.4, Nat.1)
    floor_weight_term(Nat.4, Nat.1) = Nat.1 * (Nat.4.div(Nat.1))
    nat_div_one(Nat.4)
    Nat.4.div(Nat.1) = Nat.4
    mul_one_left(Nat.4)
    Nat.1 * Nat.4 = Nat.4
    function(d: Nat) { floor_weight_term(Nat.4, d) }(Nat.1) = Nat.4
    function(d: Nat) { floor_weight_term(Nat.4, d) }(Nat.2) =
        floor_weight_term(Nat.4, Nat.2)
    floor_weight_term(Nat.4, Nat.2) = Nat.2 * (Nat.4.div(Nat.2))
    nat_div_four_two
    Nat.4.div(Nat.2) = Nat.2
    Nat.2 * Nat.2 = Nat.4
    function(d: Nat) { floor_weight_term(Nat.4, d) }(Nat.2) = Nat.4
    function(d: Nat) { floor_weight_term(Nat.4, d) }(Nat.3) =
        floor_weight_term(Nat.4, Nat.3)
    floor_weight_term(Nat.4, Nat.3) = Nat.3 * (Nat.4.div(Nat.3))
    nat_div_four_three
    Nat.4.div(Nat.3) = Nat.1
    Nat.3 * Nat.1 = Nat.3
    function(d: Nat) { floor_weight_term(Nat.4, d) }(Nat.3) = Nat.3
    function(d: Nat) { floor_weight_term(Nat.4, d) }(Nat.4) =
        floor_weight_term(Nat.4, Nat.4)
    floor_weight_term(Nat.4, Nat.4) = Nat.4 * (Nat.4.div(Nat.4))
    nat_div_self_one(Nat.4)
    Nat.4.div(Nat.4) = Nat.1
    Nat.4 * Nat.1 = Nat.4
    function(d: Nat) { floor_weight_term(Nat.4, d) }(Nat.4) = Nat.4
    range_sum(function(d: Nat) { floor_weight_term(Nat.4, d) }, Nat.1) = Nat.0
    range_sum(function(d: Nat) { floor_weight_term(Nat.4, d) }, Nat.2) = Nat.4
    range_sum(function(d: Nat) { floor_weight_term(Nat.4, d) }, Nat.3) = Nat.8
    range_sum(function(d: Nat) { floor_weight_term(Nat.4, d) }, Nat.3) +
        function(d: Nat) { floor_weight_term(Nat.4, d) }(Nat.3) = Nat.8 + Nat.3
    nat_add_8_3
    Nat.8 + Nat.3 = Nat.11
    range_sum(function(d: Nat) { floor_weight_term(Nat.4, d) }, Nat.4) = Nat.11
    range_sum(function(d: Nat) { floor_weight_term(Nat.4, d) }, Nat.4) +
        function(d: Nat) { floor_weight_term(Nat.4, d) }(Nat.4) = Nat.11 + Nat.4
    nat_add_11_4
    Nat.11 + Nat.4 = Nat.15
    range_sum(function(d: Nat) { floor_weight_term(Nat.4, d) }, Nat.5) = Nat.15
    floor_weighted_sigma_sum(Nat.4) = Nat.15
}

/// The classical identity at `n = 0`:
/// `sum_{k=1}^{0} sigma(k) = sum_{d=1}^{0} d * floor(0 / d)`.
theorem summatory_sigma_eq_floor_weighted_zero {
    summatory_sigma(Nat.0) = floor_weighted_sigma_sum(Nat.0)
} by {
    summatory_sigma_zero_value
    summatory_sigma(Nat.0) = Nat.0
    floor_weighted_sigma_sum_zero_value
    floor_weighted_sigma_sum(Nat.0) = Nat.0
    summatory_sigma(Nat.0) = floor_weighted_sigma_sum(Nat.0)
}

/// The classical identity at `n = 1`:
/// `sigma(1) = 1 * floor(1 / 1)`.
theorem summatory_sigma_eq_floor_weighted_one {
    summatory_sigma(Nat.1) = floor_weighted_sigma_sum(Nat.1)
} by {
    summatory_sigma_one_value
    summatory_sigma(Nat.1) = Nat.1
    floor_weighted_sigma_sum_one_value
    floor_weighted_sigma_sum(Nat.1) = Nat.1
    summatory_sigma(Nat.1) = floor_weighted_sigma_sum(Nat.1)
}

/// The classical identity at `n = 2`:
/// `sigma(1) + sigma(2) = 1 * floor(2 / 1) + 2 * floor(2 / 2)`.
theorem summatory_sigma_eq_floor_weighted_two {
    summatory_sigma(Nat.2) = floor_weighted_sigma_sum(Nat.2)
} by {
    summatory_sigma_two_value
    summatory_sigma(Nat.2) = Nat.4
    floor_weighted_sigma_sum_two_value
    floor_weighted_sigma_sum(Nat.2) = Nat.4
    summatory_sigma(Nat.2) = floor_weighted_sigma_sum(Nat.2)
}

/// The classical identity at `n = 3`:
/// `sigma(1) + sigma(2) + sigma(3) = 1*3 + 2*1 + 3*1`.
theorem summatory_sigma_eq_floor_weighted_three {
    summatory_sigma(Nat.3) = floor_weighted_sigma_sum(Nat.3)
} by {
    summatory_sigma_three_value
    summatory_sigma(Nat.3) = Nat.8
    floor_weighted_sigma_sum_three_value
    floor_weighted_sigma_sum(Nat.3) = Nat.8
    summatory_sigma(Nat.3) = floor_weighted_sigma_sum(Nat.3)
}

/// The classical identity at `n = 4`:
/// `sigma(1) + ... + sigma(4) = 1*4 + 2*2 + 3*1 + 4*1`.
theorem summatory_sigma_eq_floor_weighted_four {
    summatory_sigma(Nat.4) = floor_weighted_sigma_sum(Nat.4)
} by {
    summatory_sigma_four_value
    summatory_sigma(Nat.4) = Nat.15
    floor_weighted_sigma_sum_four_value
    floor_weighted_sigma_sum(Nat.4) = Nat.15
    summatory_sigma(Nat.4) = floor_weighted_sigma_sum(Nat.4)
}

// The general identity.
//
//   sum_{k=1}^{n} sigma(k) = sum_{d=1}^{n} d * floor(n / d)
//
// Expanding sigma(k) = sum_{d | k} d on the left and interchanging the two
// summations gives the sum over the pairs (k, d) with d | k and k <= n of d,
// which regrouped by d is
//
//   sum_{d=1}^{n} d * #{ k <= n : d | k } = sum_{d=1}^{n} d * floor(n / d),
//
// where #{ k <= n : d | k } = count_multiples(d, n) = n div d by
// `count_multiples_eq_div` (nat/count_multiples.ac, private to the nat
// package).  The missing formal step is the interchange of the two
// summations — a permutation argument on the pairs (k, d) with d | k — which
// is exactly the piece recorded as missing at the end of `mobius_inversion.ac`
// and `totient_sums.ac`.  The identity is verified at n = 0, 1, 2, 3, 4 above.
//
// theorem summatory_sigma_eq_floor_weighted(n: Nat) {
//     summatory_sigma(n) = floor_weighted_sigma_sum(n)
// }

// ---------------------------------------------------------------------------
// (d) The cofactor symmetry of tau.
//
// The classical manipulation
//
//     sum_{d | n} tau(d) = sum_{d | n} d * tau(n / d) * (1 / d)
//
// cancels the factor d * (1 / d) = 1 in every term, so the right-hand side is
// sum_{d | n} tau(n / d), and reindexing the divisor sum by the cofactor map
// d -> n / d (which permutes the divisors of positive n, see
// `divisor_quotient_reindex` in `totient_sums.ac`) makes it equal to
// sum_{d | n} tau(d).  The theorem below is the natural-valued form of this
// identity, stated as an equality of divisor sums.
// ---------------------------------------------------------------------------

/// The divisor sum of `tau` over the cofactors: `sum_{d | n} tau(n / d)`,
/// where the cofactor `n / d` is `divisor_quotient(n, d)`.
define cofactor_tau_divisor_sum(n: Nat) -> Nat {
    sum(map(divisor_list(n), function(d: Nat) { nat_tau(divisor_quotient(n, d)) }))
}

/// The cofactor symmetry of `tau`:
/// `sum_{d | n} tau(d) = sum_{d | n} tau(n / d)`, where the cofactor
/// `n / d` is `divisor_quotient(n, d)`.
theorem tau_divisor_sum_cofactor(n: Nat) {
    divisor_sum_fn(nat_tau)(n) = cofactor_tau_divisor_sum(n)
} by {
    if n = Nat.0 {
        divisor_list_zero
        divisor_list(Nat.0) = List.nil[Nat]
        map(List.nil[Nat], function(d: Nat) { nat_tau(divisor_quotient(Nat.0, d)) }) =
            List.nil[Nat]
        sum(List.nil[Nat]) = Nat.0
        cofactor_tau_divisor_sum(Nat.0) =
            sum(map(divisor_list(Nat.0), function(d: Nat) { nat_tau(divisor_quotient(Nat.0, d)) }))
        cofactor_tau_divisor_sum(Nat.0) = Nat.0
        divisor_sum_fn_at_zero(nat_tau)
        divisor_sum_fn(nat_tau)(Nat.0) = Nat.0
        divisor_sum_fn(nat_tau)(n) = Nat.0
        cofactor_tau_divisor_sum(n) = cofactor_tau_divisor_sum(Nat.0)
        cofactor_tau_divisor_sum(n) = Nat.0
        divisor_sum_fn(nat_tau)(n) = cofactor_tau_divisor_sum(n)
    } else {
        n != Nat.0
        Nat.0 < n
        divisor_quotient_reindex(nat_tau, n)
        sum(map(divisor_list(n), function(d: Nat) { nat_tau(divisor_quotient(n, d)) })) =
            sum(map(divisor_list(n), nat_tau))
        cofactor_tau_divisor_sum(n) =
            sum(map(divisor_list(n), function(d: Nat) { nat_tau(divisor_quotient(n, d)) }))
        cofactor_tau_divisor_sum(n) = sum(map(divisor_list(n), nat_tau))
        divisor_sum_fn_apply(nat_tau, n)
        divisor_sum_fn(nat_tau)(n) = sum(map(divisor_list(n), nat_tau))
        sum(map(divisor_list(n), nat_tau)) = cofactor_tau_divisor_sum(n)
        divisor_sum_fn(nat_tau)(n) = cofactor_tau_divisor_sum(n)
    }
}

// ---------------------------------------------------------------------------
// (e) The summatory totient.
//
// The classical asymptotic
//
//     Phi(n) = sum_{k=1}^{n} phi(k) ~ 3 n^2 / pi^2    as n -> infinity
//
// is the summatory form of Euler's totient.  A proof needs real analysis with
// the constant pi (the limit of the Dirichlet series of zeta(2) = pi^2 / 6 is
// not formalised in the library), so the asymptotic is recorded here as a
// comment and only the definition and the first values are formalised.
// ---------------------------------------------------------------------------

/// The summatory totient: `Phi(n) = sum_{k=1}^{n} phi(k)`.
///
/// As a range sum this is `range_sum(nat_totient, n.suc) = phi(0) + ... +
/// phi(n)`, and `phi(0) = 0`, so the k = 0 term contributes nothing.
define summatory_totient(n: Nat) -> Nat {
    range_sum(nat_totient, n.suc)
}

/// `Phi(0) = 0`, the empty summatory sum.
theorem summatory_totient_zero_value {
    summatory_totient(Nat.0) = Nat.0
} by {
    summatory_totient(Nat.0) = range_sum(nat_totient, Nat.1)
    range_sum_one(nat_totient)
    range_sum(nat_totient, Nat.1) = nat_totient(Nat.0)
    nat_totient_zero
    nat_totient(Nat.0) = Nat.0
    summatory_totient(Nat.0) = Nat.0
}

/// `Phi(1) = 1`, since `phi(1) = 1`.
theorem summatory_totient_one_value {
    summatory_totient(Nat.1) = Nat.1
} by {
    summatory_totient(Nat.1) = range_sum(nat_totient, Nat.2)
    range_sum_suc(nat_totient, Nat.1)
    range_sum(nat_totient, Nat.2) = range_sum(nat_totient, Nat.1) + nat_totient(Nat.1)
    range_sum_suc(nat_totient, Nat.0)
    range_sum(nat_totient, Nat.1) = range_sum(nat_totient, Nat.0) + nat_totient(Nat.0)
    range_sum_zero(nat_totient)
    range_sum(nat_totient, Nat.0) = Nat.0
    nat_totient_zero
    nat_totient(Nat.0) = Nat.0
    nat_totient_one
    nat_totient(Nat.1) = Nat.1
    range_sum(nat_totient, Nat.1) = Nat.0
    range_sum(nat_totient, Nat.2) = Nat.1
    summatory_totient(Nat.1) = Nat.1
}

// The asymptotic.
//
//   summatory_totient(n) = sum_{k=1}^{n} phi(k) ~ 3 n^2 / pi^2    (n -> infinity)
//
// The classical proof (Mertens / Dirichlet's hyperbola method) writes
// Phi(n) = sum_{d <= n} mu(d) * T(floor(n / d)) with the triangular numbers
// T(m) = m(m+1)/2, then evaluates the leading term; the constant 3 / pi^2
// comes from zeta(2) = pi^2 / 6 via sum mu(d) / d^2 = 1 / zeta(2).  A
// formalisation needs real analysis with the constant pi and the convergence
// theory of the Möbius series, which the library does not provide yet.
