from nat import Nat, mul_to_zero, divides_self, divides_mul, divides_trans,
    add_cancels_left, add_cancels_right, factorial_step, zero_or_suc
from combinatorics import binom, choose_zero, choose_one
from list import List, map, product, sum, map_add, map_singleton, product_append,
    sum_add
from number_theory.factorisation import count_prime_factor, count_prime_factor_mul,
    no_proper_divisor_imp_prime
from number_theory.legendre import prime_factor_count_upto, legendre_binom, binom_add_ne_zero
from number_theory.kummer import legendre_digit_sum, legendre_digit_sum_at,
    kummer_digit_sum_of_legendre_digit_sum
from nat import digit_sum, digit_sum_two_double

numerals Nat

/// The central binomial coefficient.
define central_binom(n: Nat) -> Nat {
    (n + n).binom(n)
}

/// The `i`th factor in the falling product from `n`.
define falling_product_factor(n: Nat, i: Nat) -> Nat {
    n - i
}

/// The prime multiplicity of the `i`th factor in the falling product from `n`.
define falling_product_factor_count(p: Nat, n: Nat, i: Nat) -> Nat {
    count_prime_factor(p, falling_product_factor(n, i))
}

/// The factors `n, n - 1, ..., n - k` as a list.
define falling_product_factors(n: Nat, k: Nat) -> List[Nat] {
    map(k.suc.range, falling_product_factor(n))
}

/// The prime multiplicities of the factors `n, n - 1, ..., n - k` as a list.
define falling_product_factor_counts(p: Nat, n: Nat, k: Nat) -> List[Nat] {
    map(k.suc.range, falling_product_factor_count(p, n))
}

/// The product `n * (n - 1) * ... * (n - k)`.
define falling_product(n: Nat, k: Nat) -> Nat {
    match k {
        Nat.zero {
            n
        }
        Nat.suc(j) {
            falling_product(n, j) * (n - k)
        }
    }
}

/// The sum of prime multiplicities in the factors of a falling product.
define falling_product_prime_count_sum(p: Nat, n: Nat, k: Nat) -> Nat {
    match k {
        Nat.zero {
            count_prime_factor(p, n)
        }
        Nat.suc(j) {
            falling_product_prime_count_sum(p, n, j) +
                count_prime_factor(p, n - k)
        }
    }
}

/// The central binomial coefficient unfolds to its defining binomial coefficient.
theorem central_binom_eq(n: Nat) {
    central_binom(n) = (n + n).binom(n)
}

/// The zeroth central binomial coefficient is one.
theorem central_binom_zero {
    central_binom(Nat.0) = Nat.1
} by {
    central_binom(Nat.0) = (Nat.0 + Nat.0).binom(Nat.0)
    Nat.0 + Nat.0 = Nat.0
    choose_zero(Nat.0)
    Nat.0.binom(Nat.0) = Nat.1
    central_binom(Nat.0) = Nat.1
}

/// The first central binomial coefficient is two.
theorem central_binom_one {
    central_binom(Nat.1) = Nat.2
} by {
    central_binom(Nat.1) = (Nat.1 + Nat.1).binom(Nat.1)
    Nat.1 + Nat.1 = Nat.2
    choose_one(Nat.2)
    Nat.2.binom(Nat.1) = Nat.2
    central_binom(Nat.1) = Nat.2
}

/// The central binomial coefficient is nonzero.
theorem central_binom_ne_zero(n: Nat) {
    central_binom(n) != Nat.0
} by {
    binom_add_ne_zero(n, n)
    central_binom(n) = (n + n).binom(n)
}

/// The zeroth falling product is the top factor.
theorem falling_product_zero(n: Nat) {
    falling_product(n, Nat.0) = n
}

/// Extending a falling product appends the next lower factor.
theorem falling_product_suc(n: Nat, k: Nat) {
    falling_product(n, k.suc) = falling_product(n, k) * (n - k.suc)
}

/// The zeroth falling-product factor list is the singleton list containing `n`.
theorem falling_product_factors_zero(n: Nat) {
    falling_product_factors(n, Nat.0) = List.singleton(n)
} by {
    falling_product_factors(n, Nat.0) =
        map(Nat.0.suc.range, falling_product_factor(n))
    Nat.0.suc.range = Nat.0.range.append(Nat.0)
    Nat.0.range = List.nil[Nat]
    Nat.0.range.append(Nat.0) = List.singleton(Nat.0)
    Nat.0.suc.range = List.singleton(Nat.0)
    map_singleton[Nat, Nat](falling_product_factor(n), Nat.0)
    map(List.singleton(Nat.0), falling_product_factor(n)) =
        List.singleton(falling_product_factor(n, Nat.0))
    falling_product_factor(n, Nat.0) = n
    falling_product_factors(n, Nat.0) = List.singleton(n)
}

/// Extending the factor list appends the next lower factor.
theorem falling_product_factors_suc(n: Nat, k: Nat) {
    falling_product_factors(n, k.suc) =
        falling_product_factors(n, k).append(n - k.suc)
} by {
    falling_product_factors(n, k.suc) =
        map(k.suc.suc.range, falling_product_factor(n))
    k.suc.suc.range = k.suc.range.append(k.suc)
    k.suc.range.append(k.suc) = k.suc.range + List.singleton(k.suc)
    map_add[Nat, Nat](k.suc.range, List.singleton(k.suc), falling_product_factor(n))
    map_singleton[Nat, Nat](falling_product_factor(n), k.suc)
    map(k.suc.suc.range, falling_product_factor(n)) =
        map(k.suc.range, falling_product_factor(n)).append(falling_product_factor(n, k.suc))
    falling_product_factor(n, k.suc) = n - k.suc
    falling_product_factors(n, k.suc) =
        falling_product_factors(n, k).append(n - k.suc)
}

/// The zeroth falling-product factor-count list is the singleton list
/// containing the valuation of `n`.
theorem falling_product_factor_counts_zero(p: Nat, n: Nat) {
    falling_product_factor_counts(p, n, Nat.0) =
        List.singleton(count_prime_factor(p, n))
} by {
    falling_product_factor_counts(p, n, Nat.0) =
        map(Nat.0.suc.range, falling_product_factor_count(p, n))
    Nat.0.suc.range = Nat.0.range.append(Nat.0)
    Nat.0.range = List.nil[Nat]
    Nat.0.range.append(Nat.0) = List.singleton(Nat.0)
    Nat.0.suc.range = List.singleton(Nat.0)
    map_singleton[Nat, Nat](falling_product_factor_count(p, n), Nat.0)
    map(List.singleton(Nat.0), falling_product_factor_count(p, n)) =
        List.singleton(falling_product_factor_count(p, n, Nat.0))
    falling_product_factor_count(p, n, Nat.0) = count_prime_factor(p, n)
    falling_product_factor_counts(p, n, Nat.0) =
        List.singleton(count_prime_factor(p, n))
}

/// Extending the factor-count list appends the valuation of the next lower
/// factor.
theorem falling_product_factor_counts_suc(p: Nat, n: Nat, k: Nat) {
    falling_product_factor_counts(p, n, k.suc) =
        falling_product_factor_counts(p, n, k).append(count_prime_factor(p, n - k.suc))
} by {
    falling_product_factor_counts(p, n, k.suc) =
        map(k.suc.suc.range, falling_product_factor_count(p, n))
    k.suc.suc.range = k.suc.range.append(k.suc)
    k.suc.range.append(k.suc) = k.suc.range + List.singleton(k.suc)
    map_add[Nat, Nat](k.suc.range, List.singleton(k.suc), falling_product_factor_count(p, n))
    map_singleton[Nat, Nat](falling_product_factor_count(p, n), k.suc)
    map(k.suc.suc.range, falling_product_factor_count(p, n)) =
        map(k.suc.range, falling_product_factor_count(p, n)).append(
            falling_product_factor_count(p, n, k.suc))
    falling_product_factor_count(p, n, k.suc) = count_prime_factor(p, n - k.suc)
    falling_product_factor_counts(p, n, k.suc) =
        falling_product_factor_counts(p, n, k).append(count_prime_factor(p, n - k.suc))
}

/// The zeroth falling-product valuation sum is the valuation of the top factor.
theorem falling_product_prime_count_sum_zero(p: Nat, n: Nat) {
    falling_product_prime_count_sum(p, n, Nat.0) = count_prime_factor(p, n)
}

/// Extending the valuation sum appends the valuation of the next factor.
theorem falling_product_prime_count_sum_suc(p: Nat, n: Nat, k: Nat) {
    falling_product_prime_count_sum(p, n, k.suc) =
        falling_product_prime_count_sum(p, n, k) + count_prime_factor(p, n - k.suc)
}

/// The recursive falling product is the product of its factor list.
theorem falling_product_eq_product_factors(n: Nat, k: Nat) {
    falling_product(n, k) = product[Nat](falling_product_factors(n, k))
} by {
    define p(j: Nat) -> Bool {
        falling_product(n, j) = product[Nat](falling_product_factors(n, j))
    }

    falling_product_zero(n)
    falling_product_factors_zero(n)
    falling_product_factors(n, Nat.0) = List.singleton(n)
    product[Nat](List.singleton(n)) = n
    falling_product(n, Nat.0) = product[Nat](falling_product_factors(n, Nat.0))
    p(Nat.0)

    forall(j: Nat) {
        if p(j) {
            falling_product_suc(n, j)
            falling_product_factors_suc(n, j)
            product_append[Nat](falling_product_factors(n, j), List.singleton(n - j.suc))
            product[Nat](List.singleton(n - j.suc)) = n - j.suc
            product[Nat](falling_product_factors(n, j).append(n - j.suc)) =
                product[Nat](falling_product_factors(n, j)) * (n - j.suc)
            product[Nat](falling_product_factors(n, j.suc)) =
                product[Nat](falling_product_factors(n, j)) * (n - j.suc)
            falling_product(n, j) = product[Nat](falling_product_factors(n, j))
            falling_product(n, j) * (n - j.suc) =
                product[Nat](falling_product_factors(n, j)) * (n - j.suc)
            falling_product(n, j.suc) =
                product[Nat](falling_product_factors(n, j)) * (n - j.suc)
            falling_product(n, j.suc) = product[Nat](falling_product_factors(n, j.suc))
            p(j.suc)
        }
    }

    p(k)
}

/// The recursive falling-product valuation sum is the sum of its
/// factor-count list.
theorem falling_product_prime_count_sum_eq_sum_factor_counts(p: Nat, n: Nat, k: Nat) {
    falling_product_prime_count_sum(p, n, k) =
        sum[Nat](falling_product_factor_counts(p, n, k))
} by {
    define q(j: Nat) -> Bool {
        falling_product_prime_count_sum(p, n, j) =
            sum[Nat](falling_product_factor_counts(p, n, j))
    }

    falling_product_prime_count_sum_zero(p, n)
    falling_product_factor_counts_zero(p, n)
    falling_product_factor_counts(p, n, Nat.0) =
        List.singleton(count_prime_factor(p, n))
    sum[Nat](List.singleton(count_prime_factor(p, n))) = count_prime_factor(p, n)
    falling_product_prime_count_sum(p, n, Nat.0) =
        sum[Nat](falling_product_factor_counts(p, n, Nat.0))
    q(Nat.0)

    forall(j: Nat) {
        if q(j) {
            let term: Nat = count_prime_factor(p, n - j.suc)
            falling_product_prime_count_sum_suc(p, n, j)
            falling_product_factor_counts_suc(p, n, j)
            sum_add[Nat](falling_product_factor_counts(p, n, j),
                List.singleton(term))
            List.singleton(term) = List.cons(term, List.nil[Nat])
            sum[Nat](List.nil[Nat]) = Nat.0
            sum[Nat](List.singleton(term)) = term + Nat.0
            term + Nat.0 = term
            sum[Nat](List.singleton(term)) = term
            sum[Nat](falling_product_factor_counts(p, n, j).append(
                count_prime_factor(p, n - j.suc))) =
                sum[Nat](falling_product_factor_counts(p, n, j)) +
                count_prime_factor(p, n - j.suc)
            sum[Nat](falling_product_factor_counts(p, n, j.suc)) =
                sum[Nat](falling_product_factor_counts(p, n, j)) +
                count_prime_factor(p, n - j.suc)
            falling_product_prime_count_sum(p, n, j) =
                sum[Nat](falling_product_factor_counts(p, n, j))
            falling_product_prime_count_sum(p, n, j) + count_prime_factor(p, n - j.suc) =
                sum[Nat](falling_product_factor_counts(p, n, j)) +
                count_prime_factor(p, n - j.suc)
            falling_product_prime_count_sum(p, n, j.suc) =
                sum[Nat](falling_product_factor_counts(p, n, j)) +
                count_prime_factor(p, n - j.suc)
            falling_product_prime_count_sum(p, n, j.suc) =
                sum[Nat](falling_product_factor_counts(p, n, j.suc))
            q(j.suc)
        }
    }

    q(k)
}

/// A falling product with all factors strictly positive is nonzero.
theorem falling_product_nonzero(n: Nat, k: Nat) {
    k < n implies falling_product(n, k) != Nat.0
} by {
    define p(j: Nat) -> Bool {
        j < n implies falling_product(n, j) != Nat.0
    }

    if Nat.0 < n {
        falling_product_zero(n)
        falling_product(n, Nat.0) = n
        if falling_product(n, Nat.0) = Nat.0 {
            n = Nat.0
            false
        }
    }
    p(Nat.0)

    forall(j: Nat) {
        if p(j) {
            if j.suc < n {
                j < n
                falling_product(n, j) != Nat.0
                n - j.suc != Nat.0
                falling_product_suc(n, j)
                falling_product(n, j.suc) = falling_product(n, j) * (n - j.suc)
                if falling_product(n, j.suc) = Nat.0 {
                    falling_product(n, j) * (n - j.suc) = Nat.0
                    mul_to_zero(falling_product(n, j), n - j.suc)
                    false
                }
            }
            p(j.suc)
        }
    }

    p(k)
}

/// The prime multiplicity of a falling product is the sum of the multiplicities
/// of its factors.
theorem count_prime_factor_falling_product(p: Nat, n: Nat, k: Nat) {
    k < n implies
        count_prime_factor(p, falling_product(n, k)) =
        falling_product_prime_count_sum(p, n, k)
} by {
    define q(j: Nat) -> Bool {
        j < n implies
            count_prime_factor(p, falling_product(n, j)) =
            falling_product_prime_count_sum(p, n, j)
    }

    if Nat.0 < n {
        falling_product_zero(n)
        falling_product_prime_count_sum_zero(p, n)
        count_prime_factor(p, falling_product(n, Nat.0)) = count_prime_factor(p, n)
        count_prime_factor(p, falling_product(n, Nat.0)) =
            falling_product_prime_count_sum(p, n, Nat.0)
    }
    q(Nat.0)

    forall(j: Nat) {
        if q(j) {
            if j.suc < n {
                j < n
                count_prime_factor(p, falling_product(n, j)) =
                    falling_product_prime_count_sum(p, n, j)
                falling_product_nonzero(n, j)
                falling_product(n, j) != Nat.0
                n - j.suc != Nat.0
                count_prime_factor_mul(p, falling_product(n, j), n - j.suc)
                count_prime_factor(p, falling_product(n, j) * (n - j.suc)) =
                    count_prime_factor(p, falling_product(n, j)) +
                    count_prime_factor(p, n - j.suc)
                falling_product_suc(n, j)
                falling_product_prime_count_sum_suc(p, n, j)
                count_prime_factor(p, falling_product(n, j.suc)) =
                    falling_product_prime_count_sum(p, n, j) +
                    count_prime_factor(p, n - j.suc)
                count_prime_factor(p, falling_product(n, j.suc)) =
                    falling_product_prime_count_sum(p, n, j.suc)
            }
            q(j.suc)
        }
    }

    q(k)
}

/// The falling product with two factors.
theorem falling_product_one(n: Nat) {
    falling_product(n, Nat.1) = n * (n - Nat.1)
} by {
    falling_product_suc(n, Nat.0)
    falling_product_zero(n)
    falling_product(n, Nat.1) = falling_product(n, Nat.0) * (n - Nat.1)
    falling_product(n, Nat.1) = n * (n - Nat.1)
}

/// The falling product with three factors.
theorem falling_product_two(n: Nat) {
    falling_product(n, Nat.2) = n * (n - Nat.1) * (n - Nat.2)
} by {
    falling_product_suc(n, Nat.1)
    falling_product_one(n)
    falling_product(n, Nat.2) = falling_product(n, Nat.1) * (n - Nat.2)
    falling_product(n, Nat.2) = n * (n - Nat.1) * (n - Nat.2)
}

/// The factor list for the falling product with two factors.
theorem falling_product_factors_one(n: Nat) {
    falling_product_factors(n, Nat.1) =
        List.cons(n, List.cons(n - Nat.1, List.nil[Nat]))
} by {
    falling_product_factors_suc(n, Nat.0)
    falling_product_factors_zero(n)
    falling_product_factors(n, Nat.1) =
        List.singleton(n).append(n - Nat.1)
    List.singleton(n) = List.cons(n, List.nil[Nat])
    List.singleton(n - Nat.1) = List.cons(n - Nat.1, List.nil[Nat])
    List.singleton(n).append(n - Nat.1) =
        List.singleton(n) + List.singleton(n - Nat.1)
    List.singleton(n) + List.singleton(n - Nat.1) =
        List.cons(n, List.cons(n - Nat.1, List.nil[Nat]))
    List.singleton(n).append(n - Nat.1) =
        List.cons(n, List.cons(n - Nat.1, List.nil[Nat]))
    falling_product_factors(n, Nat.1) =
        List.cons(n, List.cons(n - Nat.1, List.nil[Nat]))
}

/// The factor list for the falling product with three factors.
theorem falling_product_factors_two(n: Nat) {
    falling_product_factors(n, Nat.2) =
        List.cons(n, List.cons(n - Nat.1, List.cons(n - Nat.2, List.nil[Nat])))
} by {
    falling_product_factors_suc(n, Nat.1)
    falling_product_factors_one(n)
    falling_product_factors(n, Nat.2) =
        List.cons(n, List.cons(n - Nat.1, List.nil[Nat])).append(n - Nat.2)
    List.singleton(n - Nat.2) = List.cons(n - Nat.2, List.nil[Nat])
    List.cons(n, List.cons(n - Nat.1, List.nil[Nat])).append(n - Nat.2) =
        List.cons(n, List.cons(n - Nat.1, List.nil[Nat])) + List.singleton(n - Nat.2)
    List.nil[Nat] + List.singleton(n - Nat.2) = List.singleton(n - Nat.2)
    List.cons(n - Nat.1, List.nil[Nat]) + List.singleton(n - Nat.2) =
        List.cons(n - Nat.1, List.singleton(n - Nat.2))
    List.cons(n - Nat.1, List.singleton(n - Nat.2)) =
        List.cons(n - Nat.1, List.cons(n - Nat.2, List.nil[Nat]))
    List.cons(n, List.cons(n - Nat.1, List.nil[Nat])) + List.singleton(n - Nat.2) =
        List.cons(n, List.cons(n - Nat.1, List.singleton(n - Nat.2)))
    List.cons(n, List.cons(n - Nat.1, List.nil[Nat])) + List.singleton(n - Nat.2) =
        List.cons(n, List.cons(n - Nat.1, List.cons(n - Nat.2, List.nil[Nat])))
    List.cons(n, List.cons(n - Nat.1, List.nil[Nat])).append(n - Nat.2) =
        List.cons(n, List.cons(n - Nat.1, List.cons(n - Nat.2, List.nil[Nat])))
    falling_product_factors(n, Nat.2) =
        List.cons(n, List.cons(n - Nat.1, List.cons(n - Nat.2, List.nil[Nat])))
}

/// The factor-count list for the falling product with two factors.
theorem falling_product_factor_counts_one(p: Nat, n: Nat) {
    falling_product_factor_counts(p, n, Nat.1) =
        List.cons(count_prime_factor(p, n),
            List.cons(count_prime_factor(p, n - Nat.1), List.nil[Nat]))
} by {
    falling_product_factor_counts_suc(p, n, Nat.0)
    falling_product_factor_counts_zero(p, n)
    let a: Nat = count_prime_factor(p, n)
    let b: Nat = count_prime_factor(p, n - Nat.1)
    falling_product_factor_counts(p, n, Nat.1) =
        List.singleton(a).append(b)
    List.singleton(a) = List.cons(a, List.nil[Nat])
    List.singleton(b) = List.cons(b, List.nil[Nat])
    List.singleton(a).append(b) = List.singleton(a) + List.singleton(b)
    List.nil[Nat] + List.singleton(b) = List.singleton(b)
    List.cons(a, List.nil[Nat]) + List.singleton(b) = List.cons(a, List.singleton(b))
    List.cons(a, List.singleton(b)) = List.cons(a, List.cons(b, List.nil[Nat]))
    List.singleton(a) + List.singleton(b) = List.cons(a, List.cons(b, List.nil[Nat]))
    List.singleton(a).append(b) = List.cons(a, List.cons(b, List.nil[Nat]))
    falling_product_factor_counts(p, n, Nat.1) =
        List.cons(count_prime_factor(p, n),
            List.cons(count_prime_factor(p, n - Nat.1), List.nil[Nat]))
}

/// The factor-count list for the falling product with three factors.
theorem falling_product_factor_counts_two(p: Nat, n: Nat) {
    falling_product_factor_counts(p, n, Nat.2) =
        List.cons(count_prime_factor(p, n),
            List.cons(count_prime_factor(p, n - Nat.1),
                List.cons(count_prime_factor(p, n - Nat.2), List.nil[Nat])))
} by {
    falling_product_factor_counts_suc(p, n, Nat.1)
    falling_product_factor_counts_one(p, n)
    let a: Nat = count_prime_factor(p, n)
    let b: Nat = count_prime_factor(p, n - Nat.1)
    let c: Nat = count_prime_factor(p, n - Nat.2)
    falling_product_factor_counts(p, n, Nat.2) =
        List.cons(a, List.cons(b, List.nil[Nat])).append(c)
    List.singleton(c) = List.cons(c, List.nil[Nat])
    List.cons(a, List.cons(b, List.nil[Nat])).append(c) =
        List.cons(a, List.cons(b, List.nil[Nat])) + List.singleton(c)
    List.nil[Nat] + List.singleton(c) = List.singleton(c)
    List.cons(b, List.nil[Nat]) + List.singleton(c) =
        List.cons(b, List.singleton(c))
    List.cons(b, List.singleton(c)) =
        List.cons(b, List.cons(c, List.nil[Nat]))
    List.cons(a, List.cons(b, List.nil[Nat])) + List.singleton(c) =
        List.cons(a, List.cons(b, List.singleton(c)))
    List.cons(a, List.cons(b, List.nil[Nat])) + List.singleton(c) =
        List.cons(a, List.cons(b, List.cons(c, List.nil[Nat])))
    falling_product_factor_counts(p, n, Nat.2) =
        List.cons(count_prime_factor(p, n),
            List.cons(count_prime_factor(p, n - Nat.1),
                List.cons(count_prime_factor(p, n - Nat.2), List.nil[Nat])))
}

/// The valuation sum for the falling product with two factors.
theorem falling_product_prime_count_sum_one(p: Nat, n: Nat) {
    falling_product_prime_count_sum(p, n, Nat.1) =
        count_prime_factor(p, n) + count_prime_factor(p, n - Nat.1)
} by {
    falling_product_prime_count_sum_suc(p, n, Nat.0)
    falling_product_prime_count_sum_zero(p, n)
    falling_product_prime_count_sum(p, n, Nat.1) =
        falling_product_prime_count_sum(p, n, Nat.0) +
        count_prime_factor(p, n - Nat.1)
    falling_product_prime_count_sum(p, n, Nat.1) =
        count_prime_factor(p, n) + count_prime_factor(p, n - Nat.1)
}

/// The valuation sum for the falling product with three factors.
theorem falling_product_prime_count_sum_two(p: Nat, n: Nat) {
    falling_product_prime_count_sum(p, n, Nat.2) =
        count_prime_factor(p, n) +
        count_prime_factor(p, n - Nat.1) +
        count_prime_factor(p, n - Nat.2)
} by {
    falling_product_prime_count_sum_suc(p, n, Nat.1)
    falling_product_prime_count_sum_one(p, n)
    falling_product_prime_count_sum(p, n, Nat.2) =
        falling_product_prime_count_sum(p, n, Nat.1) +
        count_prime_factor(p, n - Nat.2)
    falling_product_prime_count_sum(p, n, Nat.2) =
        count_prime_factor(p, n) +
        count_prime_factor(p, n - Nat.1) +
        count_prime_factor(p, n - Nat.2)
}

/// A falling product is the initial quotient of `n!` by the remaining
/// factorial.
theorem falling_product_mul_factorial_complement(n: Nat, k: Nat) {
    k < n implies
        falling_product(n, k) * (n - k.suc).factorial = n.factorial
} by {
    define p(j: Nat) -> Bool {
        j < n implies
            falling_product(n, j) * (n - j.suc).factorial = n.factorial
    }

    if Nat.0 < n {
        zero_or_suc(n)
        let m: Nat satisfy { n = m.suc }
        n - Nat.0.suc = m
        falling_product_zero(n)
        factorial_step(m)
        n.factorial = n * (n - Nat.1).factorial
        falling_product(n, Nat.0) * (n - Nat.0.suc).factorial = n.factorial
    }
    p(Nat.0)

    forall(j: Nat) {
        if p(j) {
            if j.suc < n {
                j < n
                let ih: Bool = j < n implies falling_product(n, j) * (n - j.suc).factorial = n.factorial
                ih
                falling_product(n, j) * (n - j.suc).factorial = n.factorial
                n - j.suc != Nat.0
                j.suc.suc <= n
                let d: Nat satisfy { j.suc.suc + d = n }
                n - j.suc.suc = d
                j.suc + d.suc = n
                n - j.suc = d.suc
                (n - j.suc.suc).suc = n - j.suc
                factorial_step(n - j.suc.suc)
                (n - j.suc).factorial = (n - j.suc) * (n - j.suc.suc).factorial
                falling_product_suc(n, j)
                falling_product(n, j.suc) * (n - j.suc.suc).factorial =
                    falling_product(n, j) * (n - j.suc) *
                    (n - j.suc.suc).factorial
                falling_product(n, j) * (n - j.suc) *
                    (n - j.suc.suc).factorial =
                    falling_product(n, j) * ((n - j.suc) *
                    (n - j.suc.suc).factorial)
                falling_product(n, j.suc) * (n - j.suc.suc).factorial =
                    falling_product(n, j) * (n - j.suc).factorial
                falling_product(n, j.suc) * (n - j.suc.suc).factorial =
                    n.factorial
            }
            p(j.suc)
        }
    }

    p(k)
}

/// Every positive falling product from `n` divides `n!`.
theorem falling_product_divides_factorial(n: Nat, k: Nat) {
    k < n implies falling_product(n, k).divides(n.factorial)
} by {
    if k < n {
        falling_product_mul_factorial_complement(n, k)
        falling_product(n, k) * (n - k.suc).factorial = n.factorial
        falling_product(n, k).divides(n.factorial)
    }
}

/// Each falling product divides itself.
theorem falling_product_divides_self(n: Nat, k: Nat) {
    falling_product(n, k).divides(falling_product(n, k))
} by {
    divides_self(falling_product(n, k))
}

/// A falling product divides the product with one more factor.
theorem falling_product_divides_suc(n: Nat, k: Nat) {
    falling_product(n, k).divides(falling_product(n, k.suc))
} by {
    falling_product_suc(n, k)
    divides_self(falling_product(n, k))
    divides_mul(falling_product(n, k), falling_product(n, k), n - k.suc)
    falling_product(n, k).divides(falling_product(n, k) * (n - k.suc))
    falling_product(n, k).divides(falling_product(n, k.suc))
}

/// Shorter falling products divide longer falling products.
theorem falling_product_prefix_divides(n: Nat, j: Nat, k: Nat) {
    j <= k implies falling_product(n, j).divides(falling_product(n, k))
} by {
    define p(m: Nat) -> Bool {
        forall(i: Nat) {
            i <= m implies falling_product(n, i).divides(falling_product(n, m))
        }
    }

    forall(i: Nat) {
        if i <= Nat.0 {
            i = Nat.0
            falling_product_divides_self(n, Nat.0)
            falling_product(n, i).divides(falling_product(n, Nat.0))
        }
    }
    p(Nat.0)

    forall(m: Nat) {
        if p(m) {
            forall(i: Nat) {
                if i <= m.suc {
                    if i = m.suc {
                        falling_product_divides_self(n, m.suc)
                        falling_product(n, i).divides(falling_product(n, m.suc))
                    } else {
                        i <= m
                        falling_product(n, i).divides(falling_product(n, m))
                        falling_product_divides_suc(n, m)
                        divides_trans(falling_product(n, i), falling_product(n, m),
                            falling_product(n, m.suc))
                        falling_product(n, i).divides(falling_product(n, m.suc))
                    }
                }
            }
            p(m.suc)
        }
    }

    p(k)
}

/// Legendre's binomial valuation identity for central binomial coefficients.
theorem central_binom_legendre(p: Nat, n: Nat) {
    count_prime_factor(p, central_binom(n)) +
        prime_factor_count_upto(p, n) +
        prime_factor_count_upto(p, n) =
        prime_factor_count_upto(p, n + n)
} by {
    legendre_binom(p, n, n)
    central_binom(n) = (n + n).binom(n)
}

/// Kummer's digit-sum identity for central binomial coefficients.
theorem central_binom_kummer_digit_sum(p: Nat, n: Nat) {
    p.is_prime implies
    p * count_prime_factor(p, central_binom(n)) + digit_sum(p, n + n) =
        count_prime_factor(p, central_binom(n)) + digit_sum(p, n) + digit_sum(p, n)
} by {
    if p.is_prime {
        legendre_digit_sum(p, n)
        legendre_digit_sum_at(p, n)
        legendre_digit_sum(p, n + n)
        legendre_digit_sum_at(p, n + n)
        kummer_digit_sum_of_legendre_digit_sum(p, n, n)
        p * count_prime_factor(p, (n + n).binom(n)) + digit_sum(p, n + n) =
            count_prime_factor(p, (n + n).binom(n)) + digit_sum(p, n) + digit_sum(p, n)
        central_binom(n) = (n + n).binom(n)
        count_prime_factor(p, central_binom(n)) =
            count_prime_factor(p, (n + n).binom(n))
        p * count_prime_factor(p, central_binom(n)) + digit_sum(p, n + n) =
            p * count_prime_factor(p, (n + n).binom(n)) + digit_sum(p, n + n)
        count_prime_factor(p, central_binom(n)) + digit_sum(p, n) + digit_sum(p, n) =
            count_prime_factor(p, (n + n).binom(n)) + digit_sum(p, n) + digit_sum(p, n)
        p * count_prime_factor(p, central_binom(n)) + digit_sum(p, n + n) =
            count_prime_factor(p, central_binom(n)) + digit_sum(p, n) + digit_sum(p, n)
    }
}

/// Two is prime.
theorem nat_two_prime {
    Nat.2.is_prime
} by {
    Nat.1 < Nat.2
    forall(k: Nat) {
        if Nat.1 < k and k < Nat.2 {
            false
        }
    }
    no_proper_divisor_imp_prime(Nat.2)
}

/// The 2-adic valuation of the central binomial coefficient is the binary
/// digit sum of `n`.
theorem central_binom_two_adic_valuation(n: Nat) {
    count_prime_factor(Nat.2, central_binom(n)) = digit_sum(Nat.2, n)
} by {
    let v: Nat = count_prime_factor(Nat.2, central_binom(n))
    let s: Nat = digit_sum(Nat.2, n)
    nat_two_prime
    central_binom_kummer_digit_sum(Nat.2, n)
    digit_sum_two_double(n)
    digit_sum(Nat.2, n + n) = s
    Nat.2 * v + digit_sum(Nat.2, n + n) = v + s + s
    Nat.2 * v + s = v + s + s
    add_cancels_right(s, Nat.2 * v, v + s)
    Nat.2 * v = v + s
    Nat.2 * v = v + v
    v + v = v + s
    add_cancels_left(v, v, s)
}
