/// Deeper divisor-sum identities.
///
/// This module collects the classical identities that sit one layer above the
/// basic divisor-sum machinery of `divisor_sum.ac`:
///
///   (a) the divisor reciprocity `sum_{d | n} d = sum_{d | n} n / d`;
///   (b) the Möbius inversion of `sigma = id * 1`:
///       `sum_{d | n} mu(d) * sigma(n / d) = n`;
///   (c) the Möbius inversion of `tau = 1 * 1`:
///       `sum_{d | n} mu(d) * tau(n / d) = 1`;
///   (e) the convolution identity `sigma * 1 = id * tau`:
///       `sum_{d | n} sigma(d) = sum_{d | n} d * tau(n / d)`.
///
/// Items (a) and (e) restate results from `divisor_identities.ac` (and
/// `mobius_applications.ac` for the weighted form of (e)); items (b) and (c)
/// are the two classical Möbius inversions of the basic convolutions, proved
/// here with the general inversion theorem `mobius_inversion`.
///
/// The classical characterisation that `sigma(n)` is odd exactly when `n` is
/// a square or twice a square (item (d)) needs the parity theory of the
/// naturals (mod-2 arithmetic), which the library does not yet provide; it is
/// left as a comment at the end of the file, following `divisor_identities.ac`.
from nat import Nat
from int import Int
from list import List, map, sum, sum_map_of_pointwise
from number_theory.divisor_sum import divisor_list, divisor_sum_fn,
    divisor_sum_fn_apply, nat_sigma, nat_tau,
    divisor_sum_fn_nat_identity_arithmetic_fn_eq_sigma,
    divisor_sum_fn_nat_one_arithmetic_fn_eq_tau
from number_theory.arithmetic_functions import nat_identity_arithmetic_fn,
    nat_one_arithmetic_fn, nat_one_arithmetic_fn_apply
from number_theory.dirichlet import divisor_quotient, nat_divisor_quotient_fn,
    dirichlet_convolve
from number_theory.divisor_identities import nat_sigma_eq_divisor_sum_cofactor,
    nat_sigma_divisor_sum_eq_id_convolve_tau
from number_theory.mobius_inversion import nat_mobius
from number_theory.mobius_inversion_theorem import mobius_inversion,
    int_one_arithmetic_fn
from number_theory.mobius_applications import int_nat_sigma_fn, int_nat_identity_fn,
    id_times_tau, sigma_divisor_sum_tau_weighted, lift_from_nat, int_sum_map_from_nat
numerals Nat
numerals Int

// ---------------------------------------------------------------------------
// (a) The divisor reciprocity: sum_{d | n} d = sum_{d | n} n / d.
//
// The cofactor map `d -> n / d` permutes the divisors of positive `n`, so the
// two divisor sums agree; both equal `sigma(n)`.  (Restates
// `nat_sigma_eq_divisor_sum_cofactor` from `divisor_identities.ac`.)
// ---------------------------------------------------------------------------

/// The divisor reciprocity: `sum_{d | n} d = sum_{d | n} n / d` for positive
/// `n`, where the cofactor `n / d` is `divisor_quotient(n, d)`.
theorem divisor_sum_reciprocity(n: Nat) {
    Nat.0 < n implies
        divisor_sum_fn(nat_identity_arithmetic_fn)(n) =
        divisor_sum_fn(nat_divisor_quotient_fn(n))(n)
} by {
    if Nat.0 < n {
        nat_sigma_eq_divisor_sum_cofactor(n)
        nat_sigma(n) = divisor_sum_fn(nat_divisor_quotient_fn(n))(n)
        divisor_sum_fn_nat_identity_arithmetic_fn_eq_sigma(n)
        divisor_sum_fn(nat_identity_arithmetic_fn)(n) = nat_sigma(n)
        divisor_sum_fn(nat_identity_arithmetic_fn)(n) =
            divisor_sum_fn(nat_divisor_quotient_fn(n))(n)
    }
}

// ---------------------------------------------------------------------------
// (b) The Möbius inversion of `sigma = id * 1`.
//
// Since `sigma(n) = sum_{d | n} d` (i.e. `sigma = id * 1`), the general
// inversion theorem `mobius_inversion` at `f = id` and `g = sigma` gives
// `id = mu * sigma`, i.e. `n = sum_{d | n} mu(d) * sigma(n / d)` for positive
// `n`.  The natural-valued `sigma` values are lifted to the integers, since
// the Möbius function takes integer values.
// ---------------------------------------------------------------------------

/// `sigma = id * 1` inverted by the Möbius function:
/// `sum_{d | n} mu(d) * sigma(n / d) = n` for positive `n`, with the `sigma`
/// values lifted to the integers.
theorem sigma_mobius_inversion_id(n: Nat) {
    Nat.0 < n implies
        sum(map(divisor_list(n), function(d: Nat) {
            nat_mobius(d) * int_nat_sigma_fn(divisor_quotient(n, d))
        })) = Int.from_nat(n)
} by {
    if Nat.0 < n {
        forall(m: Nat) {
            divisor_sum_fn_nat_identity_arithmetic_fn_eq_sigma(m)
            divisor_sum_fn(nat_identity_arithmetic_fn)(m) = nat_sigma(m)
            divisor_sum_fn_apply(nat_identity_arithmetic_fn, m)
            divisor_sum_fn(nat_identity_arithmetic_fn)(m) =
                sum(map(divisor_list(m), nat_identity_arithmetic_fn))
            sum(map(divisor_list(m), nat_identity_arithmetic_fn)) = nat_sigma(m)
            int_sum_map_from_nat(divisor_list(m), nat_identity_arithmetic_fn)
            sum(map(divisor_list(m), lift_from_nat(nat_identity_arithmetic_fn))) =
                Int.from_nat(sum(map(divisor_list(m), nat_identity_arithmetic_fn)))
            Int.from_nat(sum(map(divisor_list(m), nat_identity_arithmetic_fn))) =
                Int.from_nat(nat_sigma(m))
            sum(map(divisor_list(m), lift_from_nat(nat_identity_arithmetic_fn))) =
                Int.from_nat(nat_sigma(m))
            int_nat_sigma_fn(m) = Int.from_nat(nat_sigma(m))
            sum(map(divisor_list(m), lift_from_nat(nat_identity_arithmetic_fn))) =
                int_nat_sigma_fn(m)
            forall(x: Nat) {
                if divisor_list(m).contains(x) {
                    lift_from_nat(nat_identity_arithmetic_fn)(x) =
                        Int.from_nat(nat_identity_arithmetic_fn(x))
                    nat_identity_arithmetic_fn(x) = x
                    lift_from_nat(nat_identity_arithmetic_fn)(x) = Int.from_nat(x)
                    int_nat_identity_fn(x) = Int.from_nat(x)
                    lift_from_nat(nat_identity_arithmetic_fn)(x) = int_nat_identity_fn(x)
                }
                divisor_list(m).contains(x) implies lift_from_nat(nat_identity_arithmetic_fn)(x) = int_nat_identity_fn(x)
            }
            sum_map_of_pointwise(divisor_list(m), lift_from_nat(nat_identity_arithmetic_fn),
                int_nat_identity_fn)
            sum(map(divisor_list(m), lift_from_nat(nat_identity_arithmetic_fn))) =
                sum(map(divisor_list(m), int_nat_identity_fn))
            int_nat_sigma_fn(m) = sum(map(divisor_list(m), int_nat_identity_fn))
        }
        Nat.0 < n and (forall(m: Nat) {
            int_nat_sigma_fn(m) = sum(map(divisor_list(m), int_nat_identity_fn))
        })
        mobius_inversion(int_nat_identity_fn, int_nat_sigma_fn, n)
        sum(map(divisor_list(n), function(d: Nat) {
            nat_mobius(d) * int_nat_sigma_fn(divisor_quotient(n, d))
        })) = int_nat_identity_fn(n)
        int_nat_identity_fn(n) = Int.from_nat(n)
        sum(map(divisor_list(n), function(d: Nat) {
            nat_mobius(d) * int_nat_sigma_fn(divisor_quotient(n, d))
        })) = Int.from_nat(n)
    }
}

// ---------------------------------------------------------------------------
// (c) The Möbius inversion of `tau = 1 * 1`.
//
// Since `tau(n) = sum_{d | n} 1` (i.e. `tau = 1 * 1`), the general inversion
// theorem `mobius_inversion` at `f = 1` (the constant-one integer function)
// and `g = tau` gives `1 = mu * tau`, i.e.
// `1 = sum_{d | n} mu(d) * tau(n / d)` for positive `n`.
// ---------------------------------------------------------------------------

/// The divisor-count function lifted to the integers.
define int_nat_tau_fn(n: Nat) -> Int {
    Int.from_nat(nat_tau(n))
}

/// `tau = 1 * 1` inverted by the Möbius function:
/// `sum_{d | n} mu(d) * tau(n / d) = 1` for positive `n`, with the `tau`
/// values lifted to the integers.
theorem tau_mobius_inversion_one(n: Nat) {
    Nat.0 < n implies
        sum(map(divisor_list(n), function(d: Nat) {
            nat_mobius(d) * int_nat_tau_fn(divisor_quotient(n, d))
        })) = Int.1
} by {
    if Nat.0 < n {
        forall(m: Nat) {
            divisor_sum_fn_nat_one_arithmetic_fn_eq_tau(m)
            divisor_sum_fn(nat_one_arithmetic_fn)(m) = nat_tau(m)
            divisor_sum_fn_apply(nat_one_arithmetic_fn, m)
            divisor_sum_fn(nat_one_arithmetic_fn)(m) =
                sum(map(divisor_list(m), nat_one_arithmetic_fn))
            sum(map(divisor_list(m), nat_one_arithmetic_fn)) = nat_tau(m)
            int_sum_map_from_nat(divisor_list(m), nat_one_arithmetic_fn)
            sum(map(divisor_list(m), lift_from_nat(nat_one_arithmetic_fn))) =
                Int.from_nat(sum(map(divisor_list(m), nat_one_arithmetic_fn)))
            Int.from_nat(sum(map(divisor_list(m), nat_one_arithmetic_fn))) =
                Int.from_nat(nat_tau(m))
            sum(map(divisor_list(m), lift_from_nat(nat_one_arithmetic_fn))) =
                Int.from_nat(nat_tau(m))
            int_nat_tau_fn(m) = Int.from_nat(nat_tau(m))
            sum(map(divisor_list(m), lift_from_nat(nat_one_arithmetic_fn))) =
                int_nat_tau_fn(m)
            forall(x: Nat) {
                if divisor_list(m).contains(x) {
                    lift_from_nat(nat_one_arithmetic_fn)(x) =
                        Int.from_nat(nat_one_arithmetic_fn(x))
                    nat_one_arithmetic_fn(x) = Nat.1
                    lift_from_nat(nat_one_arithmetic_fn)(x) = Int.from_nat(Nat.1)
                    Int.from_nat(Nat.1) = Int.1
                    lift_from_nat(nat_one_arithmetic_fn)(x) = Int.1
                    int_one_arithmetic_fn(x) = Int.1
                    lift_from_nat(nat_one_arithmetic_fn)(x) = int_one_arithmetic_fn(x)
                }
                divisor_list(m).contains(x) implies lift_from_nat(nat_one_arithmetic_fn)(x) = int_one_arithmetic_fn(x)
            }
            sum_map_of_pointwise(divisor_list(m), lift_from_nat(nat_one_arithmetic_fn),
                int_one_arithmetic_fn)
            sum(map(divisor_list(m), lift_from_nat(nat_one_arithmetic_fn))) =
                sum(map(divisor_list(m), int_one_arithmetic_fn))
            int_nat_tau_fn(m) = sum(map(divisor_list(m), int_one_arithmetic_fn))
        }
        Nat.0 < n and (forall(m: Nat) {
            int_nat_tau_fn(m) = sum(map(divisor_list(m), int_one_arithmetic_fn))
        })
        mobius_inversion(int_one_arithmetic_fn, int_nat_tau_fn, n)
        sum(map(divisor_list(n), function(d: Nat) {
            nat_mobius(d) * int_nat_tau_fn(divisor_quotient(n, d))
        })) = int_one_arithmetic_fn(n)
        int_one_arithmetic_fn(n) = Int.1
        sum(map(divisor_list(n), function(d: Nat) {
            nat_mobius(d) * int_nat_tau_fn(divisor_quotient(n, d))
        })) = Int.1
    }
}

// ---------------------------------------------------------------------------
// (e) The convolution identity `sigma * 1 = id * tau`.
//
// Writing `sigma = id * 1` and `tau = 1 * 1`, associativity of Dirichlet
// convolution gives `sigma * 1 = (id * 1) * 1 = id * (1 * 1) = id * tau`,
// i.e. pointwise `sum_{d | n} sigma(d) = sum_{d | n} d * tau(n / d)`.
// (Restates `sigma_divisor_sum_tau_weighted` from `mobius_applications.ac`
// and `nat_sigma_divisor_sum_eq_id_convolve_tau` from
// `divisor_identities.ac`.)
// ---------------------------------------------------------------------------

/// The convolution identity `sigma * 1 = id * tau` in the divisor-sum form:
/// `sum_{d | n} sigma(d) = sum_{d | n} d * tau(n / d)`, where the cofactor
/// `n / d` is `divisor_quotient(n, d)`.
theorem sigma_divisor_sum_eq_id_times_tau(n: Nat) {
    divisor_sum_fn(nat_sigma)(n) = id_times_tau(n)
} by {
    sigma_divisor_sum_tau_weighted(n)
    divisor_sum_fn(nat_sigma)(n) = id_times_tau(n)
}

/// The convolution identity in Dirichlet-convolution form, applied at `n`:
/// `sum_{d | n} sigma(d) = (id * tau)(n)`.
theorem sigma_divisor_sum_eq_id_convolve_tau(n: Nat) {
    divisor_sum_fn(nat_sigma)(n) =
        dirichlet_convolve(nat_identity_arithmetic_fn, nat_tau)(n)
} by {
    nat_sigma_divisor_sum_eq_id_convolve_tau(n)
}

// ---------------------------------------------------------------------------
// (d) Oddness of sigma.
//
// The classical characterisation: `sigma(n)` is odd if and only if `n` is a
// square or twice a square.  Proving it needs the parity theory of the
// naturals (mod-2 arithmetic) together with the product formula
// `sigma(n) = prod sigma(p^e)`: `sigma(p^e) = 1 + p + ... + p^e` is odd
// exactly when `p = 2` or (`p` odd and `e` even), so `sigma(n)` is odd
// exactly when every odd prime divides `n` to an even power, i.e. `n` is a
// square or twice a square.  The library has no even/odd development for the
// naturals yet, so the statement is left here as a comment rather than a
// theorem (see `divisor_identities.ac`).
//
// define nat_is_odd(n: Nat) -> Bool {
//     exists(k: Nat) { n = Nat.2 * k + Nat.1 }
// }
//
// theorem nat_sigma_odd_iff_square_or_twice_square(n: Nat) {
//     nat_is_odd(nat_sigma(n)) = (is_square(n) or exists(k: Nat) { n = Nat.2 * (k * k) })
// }
// ---------------------------------------------------------------------------
