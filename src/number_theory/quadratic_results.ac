// ---------------------------------------------------------------------------
// Classical results about quadratic residues.
//
// This file collects the classical consequences of the Legendre-symbol API:
// Euler's criterion restated in Legendre-symbol form, the two supplements
// (minus one and two), the count of the quadratic residues modulo an odd
// prime, and the closure of residues and nonresidues under multiplication.
//
// Where a result is already proved elsewhere in the library, it is restated
// here with a citation.  The genuinely new results are:
//   - the Legendre-symbol form of Euler's criterion (forward direction, and
//     the full equivalence under existence of a generator of the unit group);
//   - the square roots of one modulo an odd prime are exactly one and minus
//     one;
//   - the product of a quadratic residue and a quadratic nonresidue is a
//     quadratic nonresidue.
//
// Throughout, an odd prime is written `p = 2*h + 1`, so that
// `(p-1)/2 = h` and `p - 1` is the residue class of minus one.
// ---------------------------------------------------------------------------

from number_theory.legendre_symbol import Nat, Int, legendre_symbol,
    is_quadratic_nonresidue_mod, quadratic_nonresidue_of_not_residue,
    legendre_symbol_value_one_iff_prime_unit_quadratic_residue
from number_theory.quadratic_residue import is_quadratic_residue_mod,
    is_unit_quadratic_residue_mod, euler_criterion_unit_quadratic_residue_forward,
    quadratic_residue_coprime_iff_unit, congr_mod_preserves_coprime,
    square_coprime_imp_base, quadratic_residue_of_square_congr,
    square_mul_eq_mul_squares, quadratic_residue_mul
from number_theory.congruence import congr_mod_refl, congr_mod_symm,
    congr_mod_trans, congr_mod_mul, congr_mod_pow, congr_mod_add,
    congr_mod_zero_of_divides, mod_congr_mod_self, mod_lt
from number_theory.modular_inverse import mod_inv, mod_inv_mul_congr_one
from number_theory.quadratic_residue_supplements import prime_pred_quadratic_residue_iff_congr_one_mod_four,
    prime_pred_quadratic_residue_iff_half_even,
    prime_pred_legendre_symbol_by_mod_four,
    prime_two_quadratic_residue_iff_congr_one_or_seven_mod_eight,
    prime_two_legendre_symbol_mod_eight
from number_theory.primitive_root import is_order_double_unit_generator_mod,
    existing_order_double_unit_generator_euler_criterion_iff
from number_theory.legendre_applications import unit_qr_positive_list,
    unit_nonqr_positive_list, unit_quadratic_residue_count_half,
    unit_quadratic_nonresidue_count, nat_sq_diff, congr_le_imp_divides_diff
from number_theory.fermat import fermat_euler, prime_divides_mul
from number_theory.totient import congr_mod_add_cancel_right_pos, congr_mod_below_eq
from number_theory.factorisation import prime_imp_no_proper_divisor
from nat import sq_eq_mul, exp_mul, one_exp, add_imp_sub, add_one_right,
    alt_suc_ne_zero, mod_of_zero, small_mod, add_sub, pos_of_ne_zero,
    lt_imp_lte_suc, lt_suc, lt_trans, lte_and_lt, lte_mul, lte_mul_both,
    lte_trans, divides_self, lt_not_ref
numerals Nat
numerals Int

// ---------------------------------------------------------------------------
// Euler's criterion in Legendre-symbol form.
// ---------------------------------------------------------------------------

/// Euler's criterion, forward direction, in Legendre-symbol form: if the
/// Legendre symbol of `a` modulo the odd prime `p = 2*h + 1` is one, then
/// `a^h ≡ 1 (mod p)`, where `h = (p-1)/2`.
theorem legendre_one_imp_half_power_congr_one(p: Nat, h: Nat, a: Nat) {
    p.is_prime and p = Nat.2 * h + Nat.1 and legendre_symbol(a, p) = Int.1
        implies a.pow(h).congr_mod(Nat.1, p)
} by {
    if p.is_prime and p = Nat.2 * h + Nat.1 and legendre_symbol(a, p) = Int.1 {
        legendre_symbol_value_one_iff_prime_unit_quadratic_residue(p, a)
        (legendre_symbol(a, p) = Int.1) = is_unit_quadratic_residue_mod(a, p)
        is_unit_quadratic_residue_mod(a, p)
        euler_criterion_unit_quadratic_residue_forward(p, h, a)
        a.pow(h).congr_mod(Nat.1, p)
    }
}

/// Euler's criterion, full equivalence, in Legendre-symbol form: for `a`
/// coprime to the odd prime `p = 2*h + 1`, the Legendre symbol of `a` is one
/// exactly when `a^h ≡ 1 (mod p)`.  The converse direction uses Euler's
/// criterion, whose proof needs a generator of the unit group of `p`.
theorem legendre_one_iff_half_power_congr_one(p: Nat, h: Nat, a: Nat) {
    p.is_prime and p = Nat.2 * h + Nat.1 and
        (exists(g: Nat) { is_order_double_unit_generator_mod(g, p, h) }) and a.coprime(p)
        implies ((legendre_symbol(a, p) = Int.1) = a.pow(h).congr_mod(Nat.1, p))
} by {
    if p.is_prime and p = Nat.2 * h + Nat.1 and
            (exists(g: Nat) { is_order_double_unit_generator_mod(g, p, h) }) and a.coprime(p) {
        existing_order_double_unit_generator_euler_criterion_iff(p, h, a)
        is_quadratic_residue_mod(a, p) = a.pow(h).congr_mod(Nat.1, p)
        quadratic_residue_coprime_iff_unit(a, p)
        is_quadratic_residue_mod(a, p) = is_unit_quadratic_residue_mod(a, p)
        legendre_symbol_value_one_iff_prime_unit_quadratic_residue(p, a)
        (legendre_symbol(a, p) = Int.1) = is_unit_quadratic_residue_mod(a, p)
        (legendre_symbol(a, p) = Int.1) = a.pow(h).congr_mod(Nat.1, p)
    }
}

// ---------------------------------------------------------------------------
// The square roots of one modulo a prime.
// ---------------------------------------------------------------------------

/// The square roots of one modulo a prime are exactly one and minus one.
theorem prime_square_root_one_cases(p: Nat, z: Nat) {
    p.is_prime and z.pow(Nat.2).congr_mod(Nat.1, p)
        implies z.congr_mod(Nat.1, p) or z.congr_mod(p - Nat.1, p)
} by {
    if p.is_prime and z.pow(Nat.2).congr_mod(Nat.1, p) {
        prime_imp_no_proper_divisor(p)
        Nat.1 < p
        if p = Nat.0 {
            lt_trans(Nat.0, Nat.1, p)
            Nat.0 < Nat.0
            lt_not_ref(Nat.0)
            false
        }
        p != Nat.0
        lt_imp_lte_suc(Nat.0, p)
        Nat.1 <= p
        let r: Nat = z.mod(p)
        mod_lt(z, p)
        r < p
        mod_congr_mod_self(z, p)
        r.congr_mod(z, p)
        congr_mod_pow(r, z, p, Nat.2)
        r.pow(Nat.2).congr_mod(z.pow(Nat.2), p)
        congr_mod_trans(r.pow(Nat.2), z.pow(Nat.2), Nat.1, p)
        r.pow(Nat.2).congr_mod(Nat.1, p)
        if r = Nat.0 {
            sq_eq_mul(Nat.0)
            Nat.0.pow(Nat.2) = Nat.0 * Nat.0
            Nat.0 * Nat.0 = Nat.0
            r.pow(Nat.2) = Nat.0
            r.pow(Nat.2).congr_mod(Nat.1, p)
            Nat.0.congr_mod(Nat.1, p)
            lt_trans(Nat.0, Nat.1, p)
            Nat.0 < p
            congr_mod_below_eq(p, Nat.0, Nat.1)
            Nat.0 = Nat.1
            false
        }
        r != Nat.0
        pos_of_ne_zero(r)
        Nat.0 < r
        lt_imp_lte_suc(Nat.0, r)
        Nat.1 <= r
        lte_mul_both(Nat.1, r, r)
        Nat.1 * r <= r * r
        Nat.1 * r = r
        r <= r * r
        lte_trans(Nat.1, r, r * r)
        Nat.1 <= r * r
        sq_eq_mul(r)
        r.pow(Nat.2) = r * r
        (r * r).congr_mod(Nat.1, p)
        congr_le_imp_divides_diff(r * r, Nat.1, p)
        p.divides(r * r - Nat.1)
        nat_sq_diff(r, Nat.1)
        (r - Nat.1) * (r + Nat.1) = r * r - Nat.1
        p.divides((r - Nat.1) * (r + Nat.1))
        prime_divides_mul(p, r - Nat.1, r + Nat.1)
        p.divides(r - Nat.1) or p.divides(r + Nat.1)
        if p.divides(r - Nat.1) {
            congr_mod_zero_of_divides(p, r - Nat.1)
            (r - Nat.1).congr_mod(Nat.0, p)
            add_sub(r, Nat.1)
            r - Nat.1 + Nat.1 = r
            congr_mod_refl(Nat.1, p)
            Nat.1.congr_mod(Nat.1, p)
            congr_mod_add(r - Nat.1, Nat.1, Nat.0, Nat.1, p)
            (r - Nat.1 + Nat.1).congr_mod(Nat.0 + Nat.1, p)
            r.congr_mod(Nat.1, p)
            congr_mod_symm(r, z, p)
            z.congr_mod(r, p)
            congr_mod_trans(z, r, Nat.1, p)
            z.congr_mod(Nat.1, p)
            z.congr_mod(Nat.1, p) or z.congr_mod(p - Nat.1, p)
        }
        if p.divides(r + Nat.1) {
            congr_mod_zero_of_divides(p, r + Nat.1)
            (r + Nat.1).congr_mod(Nat.0, p)
            add_sub(p, Nat.1)
            p - Nat.1 + Nat.1 = p
            divides_self(p)
            p.divides(p)
            congr_mod_zero_of_divides(p, p)
            p.congr_mod(Nat.0, p)
            (p - Nat.1 + Nat.1).congr_mod(Nat.0, p)
            congr_mod_symm((p - Nat.1 + Nat.1), Nat.0, p)
            Nat.0.congr_mod(p - Nat.1 + Nat.1, p)
            congr_mod_trans(r + Nat.1, Nat.0, p - Nat.1 + Nat.1, p)
            (r + Nat.1).congr_mod(p - Nat.1 + Nat.1, p)
            congr_mod_add_cancel_right_pos(r, p - Nat.1, Nat.1, p)
            r.congr_mod(p - Nat.1, p)
            congr_mod_symm(r, z, p)
            z.congr_mod(r, p)
            congr_mod_trans(z, r, p - Nat.1, p)
            z.congr_mod(p - Nat.1, p)
            z.congr_mod(Nat.1, p) or z.congr_mod(p - Nat.1, p)
        }
        z.congr_mod(Nat.1, p) or z.congr_mod(p - Nat.1, p)
    }
}

/// Euler's criterion, negative case: if the Legendre symbol of `a` is minus
/// one, then `a^h ≡ -1 (mod p)`.  Together with
/// `legendre_one_iff_half_power_congr_one`, this is the full classical
/// statement `(a/p) ≡ a^((p-1)/2) (mod p)`; it needs a generator of the unit
/// group for the converse of Euler's criterion.
theorem legendre_neg_one_imp_half_power_congr_pred(p: Nat, h: Nat, a: Nat) {
    p.is_prime and p = Nat.2 * h + Nat.1 and
        (exists(g: Nat) { is_order_double_unit_generator_mod(g, p, h) }) and a.coprime(p)
        and legendre_symbol(a, p) = -Int.1
        implies a.pow(h).congr_mod(p - Nat.1, p)
} by {
    if p.is_prime and p = Nat.2 * h + Nat.1 and
            (exists(g: Nat) { is_order_double_unit_generator_mod(g, p, h) }) and a.coprime(p)
            and legendre_symbol(a, p) = -Int.1 {
        legendre_one_iff_half_power_congr_one(p, h, a)
        (legendre_symbol(a, p) = Int.1) = a.pow(h).congr_mod(Nat.1, p)
        if a.pow(h).congr_mod(Nat.1, p) {
            legendre_symbol(a, p) = Int.1
            Int.1 = -Int.1
            false
        }
        not a.pow(h).congr_mod(Nat.1, p)
        fermat_euler(p, a)
        a.pow(p - Nat.1).congr_mod(Nat.1, p)
        add_imp_sub(Nat.2 * h, Nat.1, p)
        p - Nat.1 = Nat.2 * h
        exp_mul(a, h, Nat.2)
        a.pow(h * Nat.2) = a.pow(h).pow(Nat.2)
        h * Nat.2 = Nat.2 * h
        a.pow(Nat.2 * h) = a.pow(h).pow(Nat.2)
        a.pow(p - Nat.1) = a.pow(h).pow(Nat.2)
        a.pow(h).pow(Nat.2).congr_mod(Nat.1, p)
        prime_square_root_one_cases(p, a.pow(h))
        a.pow(h).congr_mod(Nat.1, p) or a.pow(h).congr_mod(p - Nat.1, p)
        if a.pow(h).congr_mod(Nat.1, p) {
            false
        }
        a.pow(h).congr_mod(p - Nat.1, p)
    }
}

// ---------------------------------------------------------------------------
// The first supplement: minus one.
// ---------------------------------------------------------------------------

/// Minus one is a quadratic residue modulo the odd prime `p = 2*h + 1` exactly
/// when the half-exponent `h = (p-1)/2` is even.  This is
/// `prime_pred_quadratic_residue_iff_half_even` from
/// `quadratic_residue_supplements.ac`, restated; combined with
/// `half_pred_even_iff_congr_one_mod_four` from the same file, it yields the
/// classical form `p ≡ 1 (mod 4)`.
theorem neg_one_quadratic_residue_iff_half_even(p: Nat, h: Nat) {
    p.is_prime and p = Nat.2 * h + Nat.1 implies
        (is_quadratic_residue_mod(p - Nat.1, p) = Nat.2.divides(h))
} by {
    if p.is_prime and p = Nat.2 * h + Nat.1 {
        prime_pred_quadratic_residue_iff_half_even(p, h)
        is_quadratic_residue_mod(p - Nat.1, p) = Nat.2.divides(h)
    }
}

/// Minus one is a quadratic residue modulo the odd prime `p = 2*h + 1` exactly
/// when `p` is congruent to one modulo four.  This is
/// `prime_pred_quadratic_residue_iff_congr_one_mod_four` from
/// `quadratic_residue_supplements.ac`, restated.
theorem neg_one_quadratic_residue_iff_congr_one_mod_four(p: Nat, h: Nat) {
    p.is_prime and p = Nat.2 * h + Nat.1 implies
        (is_quadratic_residue_mod(p - Nat.1, p) = p.congr_mod(Nat.1, Nat.4))
} by {
    if p.is_prime and p = Nat.2 * h + Nat.1 {
        prime_pred_quadratic_residue_iff_congr_one_mod_four(p, h)
        is_quadratic_residue_mod(p - Nat.1, p) = p.congr_mod(Nat.1, Nat.4)
    }
}

/// The Legendre symbol of minus one modulo the odd prime `p = 2*h + 1` is one
/// precisely in the residue class one modulo four, and minus one otherwise.
/// This is `prime_pred_legendre_symbol_by_mod_four`, restated.
theorem legendre_symbol_neg_one_iff_congr_one_mod_four(p: Nat, h: Nat) {
    p.is_prime and p = Nat.2 * h + Nat.1 implies
        legendre_symbol(p - Nat.1, p) =
            if p.congr_mod(Nat.1, Nat.4) { Int.1 } else { -Int.1 }
} by {
    if p.is_prime and p = Nat.2 * h + Nat.1 {
        prime_pred_legendre_symbol_by_mod_four(p, h)
        legendre_symbol(p - Nat.1, p) =
            if p.congr_mod(Nat.1, Nat.4) { Int.1 } else { -Int.1 }
    }
}

// ---------------------------------------------------------------------------
// The second supplement: two.
// ---------------------------------------------------------------------------

/// Two is a quadratic residue modulo the odd prime `p = 2*h + 1` exactly in
/// the residue classes one and seven modulo eight.  This is
/// `prime_two_quadratic_residue_iff_congr_one_or_seven_mod_eight` from
/// `quadratic_residue_supplements.ac`, restated; like the converse of Euler's
/// criterion it carries the hypothesis that the unit group of `p` has a
/// generator.
theorem two_quadratic_residue_iff_congr_one_or_seven_mod_eight(p: Nat, h: Nat) {
    p.is_prime and p = Nat.2 * h + Nat.1 and
        (exists(g: Nat) { is_order_double_unit_generator_mod(g, p, h) })
    implies (is_quadratic_residue_mod(Nat.2, p) =
        (p.congr_mod(Nat.1, Nat.8) or p.congr_mod(Nat.7, Nat.8)))
} by {
    if p.is_prime and p = Nat.2 * h + Nat.1 and
            (exists(g: Nat) { is_order_double_unit_generator_mod(g, p, h) }) {
        prime_two_quadratic_residue_iff_congr_one_or_seven_mod_eight(p, h)
        is_quadratic_residue_mod(Nat.2, p) =
            (p.congr_mod(Nat.1, Nat.8) or p.congr_mod(Nat.7, Nat.8))
    }
}

/// The Legendre symbol of two modulo the odd prime `p = 2*h + 1` is one in
/// the residue classes one and seven modulo eight and minus one otherwise.
/// This is `prime_two_legendre_symbol_mod_eight`, restated.
theorem legendre_symbol_two_iff_congr_one_or_seven_mod_eight(p: Nat, h: Nat) {
    p.is_prime and p = Nat.2 * h + Nat.1 and
        (exists(g: Nat) { is_order_double_unit_generator_mod(g, p, h) })
    implies legendre_symbol(Nat.2, p) =
        if p.congr_mod(Nat.1, Nat.8) or p.congr_mod(Nat.7, Nat.8) {
            Int.1
        } else {
            -Int.1
        }
} by {
    if p.is_prime and p = Nat.2 * h + Nat.1 and
            (exists(g: Nat) { is_order_double_unit_generator_mod(g, p, h) }) {
        prime_two_legendre_symbol_mod_eight(p, h)
        legendre_symbol(Nat.2, p) =
            if p.congr_mod(Nat.1, Nat.8) or p.congr_mod(Nat.7, Nat.8) {
                Int.1
            } else {
                -Int.1
            }
    }
}

// ---------------------------------------------------------------------------
// The count of the quadratic residues modulo an odd prime.
// ---------------------------------------------------------------------------

/// The number of unit quadratic residues in `1, ..., p-1` is `(p-1)/2`.  This
/// is `unit_quadratic_residue_count_half` from `legendre_applications.ac`,
/// restated.
theorem number_of_quadratic_residues(p: Nat, h: Nat) {
    p.is_prime and p = Nat.2 * h + Nat.1 implies
        unit_qr_positive_list(p).length = (p - Nat.1).div(Nat.2)
} by {
    if p.is_prime and p = Nat.2 * h + Nat.1 {
        unit_quadratic_residue_count_half(p, h)
        unit_qr_positive_list(p).length = (p - Nat.1).div(Nat.2)
    }
}

/// The number of unit quadratic nonresidues in `1, ..., p-1` is also `(p-1)/2`.
/// This is `unit_quadratic_nonresidue_count` from
/// `legendre_applications.ac`, restated.
theorem number_of_quadratic_nonresidues(p: Nat, h: Nat) {
    p.is_prime and p = Nat.2 * h + Nat.1 implies
        unit_nonqr_positive_list(p).length = h
} by {
    if p.is_prime and p = Nat.2 * h + Nat.1 {
        unit_quadratic_nonresidue_count(p, h)
        unit_nonqr_positive_list(p).length = h
    }
}

// ---------------------------------------------------------------------------
// Closure under multiplication.
// ---------------------------------------------------------------------------

/// The product of two quadratic residues modulo `p` is a quadratic residue.
theorem product_of_quadratic_residues_is_residue(a: Nat, b: Nat, p: Nat) {
    is_quadratic_residue_mod(a, p) and is_quadratic_residue_mod(b, p)
        implies is_quadratic_residue_mod(a * b, p)
} by {
    if is_quadratic_residue_mod(a, p) and is_quadratic_residue_mod(b, p) {
        quadratic_residue_mul(a, b, p)
        is_quadratic_residue_mod(a * b, p)
    }
}

/// The product of a quadratic residue and a quadratic nonresidue modulo the
/// prime `p` is a quadratic nonresidue.  The residue is required to be a unit:
/// otherwise (for `a ≡ 0`) the product would be a residue.
theorem quadratic_residue_mul_nonresidue(p: Nat, a: Nat, b: Nat) {
    p.is_prime and a.coprime(p) and is_quadratic_residue_mod(a, p) and
        is_quadratic_nonresidue_mod(b, p)
        implies is_quadratic_nonresidue_mod(a * b, p)
} by {
    if p.is_prime and a.coprime(p) and is_quadratic_residue_mod(a, p) and
            is_quadratic_nonresidue_mod(b, p) {
        if is_quadratic_residue_mod(a * b, p) {
            let x: Nat satisfy { x.pow(Nat.2).congr_mod(a, p) }
            let y: Nat satisfy { y.pow(Nat.2).congr_mod(a * b, p) }
            congr_mod_symm(x.pow(Nat.2), a, p)
            a.congr_mod(x.pow(Nat.2), p)
            congr_mod_preserves_coprime(a, x.pow(Nat.2), p)
            x.pow(Nat.2).coprime(p)
            square_coprime_imp_base(x, p)
            x.coprime(p)
            mod_inv_mul_congr_one(x, p)
            (x * mod_inv(x, p)).congr_mod(Nat.1, p)
            congr_mod_pow(x * mod_inv(x, p), Nat.1, p, Nat.2)
            (x * mod_inv(x, p)).pow(Nat.2).congr_mod(Nat.1.pow(Nat.2), p)
            one_exp(Nat.2)
            Nat.1.pow(Nat.2) = Nat.1
            (x * mod_inv(x, p)).pow(Nat.2).congr_mod(Nat.1, p)
            square_mul_eq_mul_squares(x, mod_inv(x, p))
            (x * mod_inv(x, p)).pow(Nat.2) =
                x.pow(Nat.2) * mod_inv(x, p).pow(Nat.2)
            (x.pow(Nat.2) * mod_inv(x, p).pow(Nat.2)).congr_mod(Nat.1, p)
            congr_mod_refl(mod_inv(x, p).pow(Nat.2), p)
            mod_inv(x, p).pow(Nat.2).congr_mod(mod_inv(x, p).pow(Nat.2), p)
            congr_mod_mul(x.pow(Nat.2), mod_inv(x, p).pow(Nat.2), a,
                mod_inv(x, p).pow(Nat.2), p)
            (x.pow(Nat.2) * mod_inv(x, p).pow(Nat.2)).congr_mod(
                a * mod_inv(x, p).pow(Nat.2), p)
            congr_mod_symm((x.pow(Nat.2) * mod_inv(x, p).pow(Nat.2)),
                a * mod_inv(x, p).pow(Nat.2), p)
            (a * mod_inv(x, p).pow(Nat.2)).congr_mod(
                x.pow(Nat.2) * mod_inv(x, p).pow(Nat.2), p)
            congr_mod_trans((a * mod_inv(x, p).pow(Nat.2)),
                x.pow(Nat.2) * mod_inv(x, p).pow(Nat.2), Nat.1, p)
            (a * mod_inv(x, p).pow(Nat.2)).congr_mod(Nat.1, p)
            congr_mod_refl(b, p)
            b.congr_mod(b, p)
            congr_mod_mul(a * mod_inv(x, p).pow(Nat.2), b, Nat.1, b, p)
            ((a * mod_inv(x, p).pow(Nat.2)) * b).congr_mod(Nat.1 * b, p)
            Nat.1 * b = b
            ((a * mod_inv(x, p).pow(Nat.2)) * b).congr_mod(b, p)
            (a * b) * mod_inv(x, p).pow(Nat.2) =
                (a * mod_inv(x, p).pow(Nat.2)) * b
            ((a * b) * mod_inv(x, p).pow(Nat.2)).congr_mod(b, p)
            congr_mod_mul(y.pow(Nat.2), mod_inv(x, p).pow(Nat.2), a * b,
                mod_inv(x, p).pow(Nat.2), p)
            (y.pow(Nat.2) * mod_inv(x, p).pow(Nat.2)).congr_mod(
                (a * b) * mod_inv(x, p).pow(Nat.2), p)
            square_mul_eq_mul_squares(y, mod_inv(x, p))
            (y * mod_inv(x, p)).pow(Nat.2) =
                y.pow(Nat.2) * mod_inv(x, p).pow(Nat.2)
            (y * mod_inv(x, p)).pow(Nat.2).congr_mod(b, p)
            quadratic_residue_of_square_congr(y * mod_inv(x, p), b, p)
            is_quadratic_residue_mod(b, p)
            is_quadratic_nonresidue_mod(b, p) =
                not is_quadratic_residue_mod(b, p)
            not is_quadratic_residue_mod(b, p)
            false
        }
        not is_quadratic_residue_mod(a * b, p)
        quadratic_nonresidue_of_not_residue(a * b, p)
        is_quadratic_nonresidue_mod(a * b, p)
    }
}
