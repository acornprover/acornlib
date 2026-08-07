from nat import Nat, add_to_zero, lt_mul_both, lt_not_ref, mul_one_left, mul_one_right,
    mul_zero_right, pos_of_ne_zero
from rat import Rat, add_zero_right, zero_recip, from_nat_add
from list import List
from data.list.list_prefix_suffix import list_take, list_take_zero, list_take_nil,
    list_take_cons_suc, list_take_append_drop, list_take_length
from pair import Pair, pair_ext, pair_new_first, pair_new_second

numerals Nat

/// True if every coefficient in a continued-fraction tail is positive.
define positive_continued_fraction_tail(coefficients: List[Nat]) -> Bool {
    match coefficients {
        List.nil {
            true
        }
        List.cons(head, tail) {
            Nat.0 < head and positive_continued_fraction_tail(tail)
        }
    }
}

/// True if the coefficients form a finite simple continued fraction.
define finite_continued_fraction_coefficients(coefficients: List[Nat]) -> Bool {
    match coefficients {
        List.nil {
            false
        }
        List.cons(head, tail) {
            positive_continued_fraction_tail(tail)
        }
    }
}

/// A finite simple continued fraction.
structure ContinuedFraction {
    /// The coefficients of the continued fraction.
    coefficients: List[Nat]
} constraint {
    finite_continued_fraction_coefficients(coefficients)
}

/// The rational value of a finite simple continued fraction.
define continued_fraction_value(coefficients: List[Nat]) -> Rat {
    match coefficients {
        List.nil {
            Rat.0
        }
        List.cons(head, tail) {
            Rat.from_nat(head) + continued_fraction_value(tail).inverse
        }
    }
}

/// The forward recurrence state for a finite continuant.
define continuant_state(coefficients: List[Nat], previous: Nat, current: Nat) -> Nat {
    match coefficients {
        List.nil {
            current
        }
        List.cons(head, tail) {
            continuant_state(tail, current, current * head + previous)
        }
    }
}

/// The continuant associated to a finite coefficient list.
define continuant(coefficients: List[Nat]) -> Nat {
    continuant_state(coefficients, Nat.0, Nat.1)
}

/// The numerator of the finite continued-fraction convergent.
define continued_fraction_numerator(coefficients: List[Nat]) -> Nat {
    match coefficients {
        List.nil {
            Nat.0
        }
        List.cons(head, tail) {
            continuant(coefficients)
        }
    }
}

/// The denominator of the finite continued-fraction convergent.
define continued_fraction_denominator(coefficients: List[Nat]) -> Nat {
    match coefficients {
        List.nil {
            Nat.1
        }
        List.cons(head, tail) {
            continuant(tail)
        }
    }
}

/// The numerator-denominator recurrence state for finite convergents.
define continued_fraction_convergent_state(coefficients: List[Nat],
    previous_numerator: Nat, current_numerator: Nat,
    previous_denominator: Nat, current_denominator: Nat) -> Pair[Nat, Nat] {
    match coefficients {
        List.nil {
            Pair.new(current_numerator, current_denominator)
        }
        List.cons(head, tail) {
            continued_fraction_convergent_state(tail,
                current_numerator,
                current_numerator * head + previous_numerator,
                current_denominator,
                current_denominator * head + previous_denominator)
        }
    }
}

/// The numerator-denominator pair for a finite continued fraction.
define continued_fraction_convergent(coefficients: List[Nat]) -> Pair[Nat, Nat] {
    match coefficients {
        List.nil {
            Pair.new(Nat.0, Nat.1)
        }
        List.cons(head, tail) {
            continued_fraction_convergent_state(coefficients, Nat.0, Nat.1, Nat.1, Nat.0)
        }
    }
}

/// The first `n` coefficients of a coefficient list.
define continued_fraction_prefix_coefficients(
    coefficients: List[Nat], n: Nat
) -> List[Nat] {
    list_take(coefficients, n)
}

/// The coefficients left after removing the first `n` coefficients.
define continued_fraction_suffix_coefficients(
    coefficients: List[Nat], n: Nat
) -> List[Nat] {
    coefficients.drop(n)
}

/// True when the first `n` coefficients form a finite simple continued
/// fraction.
define finite_continued_fraction_prefix_coefficients(
    coefficients: List[Nat], n: Nat
) -> Bool {
    finite_continued_fraction_coefficients(
        continued_fraction_prefix_coefficients(coefficients, n))
}

/// True when the coefficients after removing the first `n` coefficients form a
/// finite simple continued fraction.
define finite_continued_fraction_suffix_coefficients(
    coefficients: List[Nat], n: Nat
) -> Bool {
    finite_continued_fraction_coefficients(
        continued_fraction_suffix_coefficients(coefficients, n))
}

/// True when the first `n` coefficients have positive tail.
define positive_continued_fraction_prefix_tail(
    coefficients: List[Nat], n: Nat
) -> Bool {
    positive_continued_fraction_tail(
        continued_fraction_prefix_coefficients(coefficients, n))
}

/// True when the coefficients after removing the first `n` coefficients have
/// positive tail.
define positive_continued_fraction_suffix_tail(
    coefficients: List[Nat], n: Nat
) -> Bool {
    positive_continued_fraction_tail(
        continued_fraction_suffix_coefficients(coefficients, n))
}

/// The rational value of the first `n` coefficients.
define continued_fraction_prefix_value(coefficients: List[Nat], n: Nat) -> Rat {
    continued_fraction_value(continued_fraction_prefix_coefficients(coefficients, n))
}

/// The rational value after removing the first `n` coefficients.
define continued_fraction_suffix_value(coefficients: List[Nat], n: Nat) -> Rat {
    continued_fraction_value(continued_fraction_suffix_coefficients(coefficients, n))
}

/// The continuant of the first `n` coefficients.
define continued_fraction_prefix_continuant(coefficients: List[Nat], n: Nat) -> Nat {
    continuant(continued_fraction_prefix_coefficients(coefficients, n))
}

/// The continuant after removing the first `n` coefficients.
define continued_fraction_suffix_continuant(coefficients: List[Nat], n: Nat) -> Nat {
    continuant(continued_fraction_suffix_coefficients(coefficients, n))
}

/// The numerator of the first `n` coefficients.
define continued_fraction_prefix_numerator(coefficients: List[Nat], n: Nat) -> Nat {
    continued_fraction_numerator(continued_fraction_prefix_coefficients(coefficients, n))
}

/// The numerator after removing the first `n` coefficients.
define continued_fraction_suffix_numerator(coefficients: List[Nat], n: Nat) -> Nat {
    continued_fraction_numerator(
        continued_fraction_suffix_coefficients(coefficients, n))
}

/// The denominator of the first `n` coefficients.
define continued_fraction_prefix_denominator(
    coefficients: List[Nat], n: Nat
) -> Nat {
    continued_fraction_denominator(
        continued_fraction_prefix_coefficients(coefficients, n))
}

/// The denominator after removing the first `n` coefficients.
define continued_fraction_suffix_denominator(
    coefficients: List[Nat], n: Nat
) -> Nat {
    continued_fraction_denominator(
        continued_fraction_suffix_coefficients(coefficients, n))
}

/// The convergent pair of the first `n` coefficients.
define continued_fraction_prefix_convergent(
    coefficients: List[Nat], n: Nat
) -> Pair[Nat, Nat] {
    continued_fraction_convergent(
        continued_fraction_prefix_coefficients(coefficients, n))
}

/// The convergent pair after removing the first `n` coefficients.
define continued_fraction_suffix_convergent(
    coefficients: List[Nat], n: Nat
) -> Pair[Nat, Nat] {
    continued_fraction_convergent(
        continued_fraction_suffix_coefficients(coefficients, n))
}

/// The empty tail is positive.
theorem positive_continued_fraction_tail_nil {
    positive_continued_fraction_tail(List.nil[Nat])
}

/// The head of a positive continued-fraction tail is positive.
theorem positive_continued_fraction_tail_cons_head(head: Nat, tail: List[Nat]) {
    positive_continued_fraction_tail(List.cons(head, tail)) implies Nat.0 < head
}

/// The tail of a positive continued-fraction tail is positive.
theorem positive_continued_fraction_tail_cons_tail(head: Nat, tail: List[Nat]) {
    positive_continued_fraction_tail(List.cons(head, tail))
        implies positive_continued_fraction_tail(tail)
}

/// Consing a positive coefficient onto a positive tail gives a positive tail.
theorem positive_continued_fraction_tail_cons_intro(head: Nat, tail: List[Nat]) {
    Nat.0 < head and positive_continued_fraction_tail(tail)
        implies positive_continued_fraction_tail(List.cons(head, tail))
}

/// A singleton tail is positive exactly when its coefficient is positive.
theorem positive_continued_fraction_tail_singleton_intro(head: Nat) {
    Nat.0 < head implies positive_continued_fraction_tail(List.cons(head, List.nil[Nat]))
} by {
    if Nat.0 < head {
        positive_continued_fraction_tail_nil
        positive_continued_fraction_tail_cons_intro(head, List.nil[Nat])
    }
}

/// The coefficient in a positive singleton tail is positive.
theorem positive_continued_fraction_tail_singleton_positive(head: Nat) {
    positive_continued_fraction_tail(List.cons(head, List.nil[Nat])) implies Nat.0 < head
} by {
    if positive_continued_fraction_tail(List.cons(head, List.nil[Nat])) {
        positive_continued_fraction_tail_cons_head(head, List.nil[Nat])
    }
}

/// A two-element tail is positive when both coefficients are positive.
theorem positive_continued_fraction_tail_pair_intro(head: Nat, next: Nat) {
    Nat.0 < head and Nat.0 < next
        implies positive_continued_fraction_tail(List.cons(head, List.cons(next, List.nil[Nat])))
} by {
    if Nat.0 < head and Nat.0 < next {
        positive_continued_fraction_tail_nil
        positive_continued_fraction_tail_cons_intro(next, List.nil[Nat])
        positive_continued_fraction_tail(List.cons(next, List.nil[Nat]))
        positive_continued_fraction_tail_cons_intro(head, List.cons(next, List.nil[Nat]))
    }
}

/// True if appending a fixed positive tail to a positive tail preserves positivity.
define positive_continued_fraction_tail_append_pred(
    right: List[Nat], left: List[Nat]
) -> Bool {
    positive_continued_fraction_tail(left) and positive_continued_fraction_tail(right)
        implies positive_continued_fraction_tail(left + right)
}

/// Appending a fixed positive continued-fraction tail preserves positivity of
/// every positive tail.
theorem positive_continued_fraction_tail_append_all(right: List[Nat]) {
    forall(left: List[Nat]) {
        positive_continued_fraction_tail_append_pred(right, left)
    }
} by {
    if positive_continued_fraction_tail(List.nil[Nat])
        and positive_continued_fraction_tail(right) {
        positive_continued_fraction_tail(List.nil[Nat] + right)
    }
    positive_continued_fraction_tail_append_pred(right, List.nil[Nat])
    forall(head: Nat, tail: List[Nat]) {
        if positive_continued_fraction_tail_append_pred(right, tail) {
            if positive_continued_fraction_tail(List.cons(head, tail))
                and positive_continued_fraction_tail(right) {
                positive_continued_fraction_tail_cons_head(head, tail)
                positive_continued_fraction_tail_cons_tail(head, tail)
                positive_continued_fraction_tail_append_pred(right, tail) =
                    (positive_continued_fraction_tail(tail)
                        and positive_continued_fraction_tail(right)
                        implies positive_continued_fraction_tail(tail + right))
                let tail_positive: Bool = positive_continued_fraction_tail_append_pred(right, tail)
                positive_continued_fraction_tail_cons_intro(head, tail + right)
                positive_continued_fraction_tail(List.cons(head, tail) + right)
            }
            positive_continued_fraction_tail_append_pred(right, List.cons(head, tail))
        }
        positive_continued_fraction_tail_append_pred(right, tail) implies positive_continued_fraction_tail_append_pred(right, List.cons(head, tail))
    }
    List.induction(function(left: List[Nat]) {
        positive_continued_fraction_tail_append_pred(right, left)
    })
}

/// Appending two positive continued-fraction tails gives a positive tail.
theorem positive_continued_fraction_tail_append(left: List[Nat], right: List[Nat]) {
    positive_continued_fraction_tail(left) and positive_continued_fraction_tail(right)
        implies positive_continued_fraction_tail(left + right)
} by {
    positive_continued_fraction_tail_append_all(right)
}

/// True if positivity of an append implies positivity of its left part.
define positive_continued_fraction_tail_append_left_pred(
    right: List[Nat], left: List[Nat]
) -> Bool {
    positive_continued_fraction_tail(left + right)
        implies positive_continued_fraction_tail(left)
}

/// Positivity of an append implies positivity of every fixed left part.
theorem positive_continued_fraction_tail_append_left_all(right: List[Nat]) {
    forall(left: List[Nat]) {
        positive_continued_fraction_tail_append_left_pred(right, left)
    }
} by {
    if positive_continued_fraction_tail(List.nil[Nat] + right) {
        positive_continued_fraction_tail_nil
    }
    positive_continued_fraction_tail_append_left_pred(right, List.nil[Nat])
    forall(head: Nat, tail: List[Nat]) {
        if positive_continued_fraction_tail_append_left_pred(right, tail) {
            if positive_continued_fraction_tail(List.cons(head, tail) + right) {
                positive_continued_fraction_tail(List.cons(head, tail + right))
                positive_continued_fraction_tail_cons_head(head, tail + right)
                positive_continued_fraction_tail_cons_tail(head, tail + right)
                let tail_positive: Bool =
                    positive_continued_fraction_tail_append_left_pred(right, tail)
                positive_continued_fraction_tail(tail)
                positive_continued_fraction_tail_cons_intro(head, tail)
                positive_continued_fraction_tail(List.cons(head, tail))
            }
            positive_continued_fraction_tail_append_left_pred(right, List.cons(head, tail))
        }
        positive_continued_fraction_tail_append_left_pred(right, tail) implies positive_continued_fraction_tail_append_left_pred(right, List.cons(head, tail))
    }
    List.induction(function(left: List[Nat]) {
        positive_continued_fraction_tail_append_left_pred(right, left)
    })
}

/// The left side of an appended positive tail is positive.
theorem positive_continued_fraction_tail_append_left(left: List[Nat], right: List[Nat]) {
    positive_continued_fraction_tail(left + right)
        implies positive_continued_fraction_tail(left)
} by {
    positive_continued_fraction_tail_append_left_all(right)
}

/// True if positivity of an append implies positivity of its right part.
define positive_continued_fraction_tail_append_right_pred(
    right: List[Nat], left: List[Nat]
) -> Bool {
    positive_continued_fraction_tail(left + right)
        implies positive_continued_fraction_tail(right)
}

/// Positivity of an append implies positivity of the fixed right part.
theorem positive_continued_fraction_tail_append_right_all(right: List[Nat]) {
    forall(left: List[Nat]) {
        positive_continued_fraction_tail_append_right_pred(right, left)
    }
} by {
    if positive_continued_fraction_tail(List.nil[Nat] + right) {
        positive_continued_fraction_tail(right)
    }
    positive_continued_fraction_tail_append_right_pred(right, List.nil[Nat])
    forall(head: Nat, tail: List[Nat]) {
        if positive_continued_fraction_tail_append_right_pred(right, tail) {
            if positive_continued_fraction_tail(List.cons(head, tail) + right) {
                positive_continued_fraction_tail_cons_tail(head, tail + right)
                positive_continued_fraction_tail(tail + right)
                let tail_positive: Bool =
                    positive_continued_fraction_tail_append_right_pred(right, tail)
                positive_continued_fraction_tail(right)
            }
            positive_continued_fraction_tail_append_right_pred(right, List.cons(head, tail))
        }
        positive_continued_fraction_tail_append_right_pred(right, tail) implies positive_continued_fraction_tail_append_right_pred(right, List.cons(head, tail))
    }
    List.induction(function(left: List[Nat]) {
        positive_continued_fraction_tail_append_right_pred(right, left)
    })
}

/// The right side of an appended positive tail is positive.
theorem positive_continued_fraction_tail_append_right(left: List[Nat], right: List[Nat]) {
    positive_continued_fraction_tail(left + right)
        implies positive_continued_fraction_tail(right)
} by {
    positive_continued_fraction_tail_append_right_all(right)
}

/// An appended tail is positive exactly when both parts are positive.
theorem positive_continued_fraction_tail_append_iff(left: List[Nat], right: List[Nat]) {
    positive_continued_fraction_tail(left + right) =
        (positive_continued_fraction_tail(left) and positive_continued_fraction_tail(right))
} by {
    if positive_continued_fraction_tail(left + right) {
        positive_continued_fraction_tail_append_left(left, right)
        positive_continued_fraction_tail_append_right(left, right)
        positive_continued_fraction_tail(left) and positive_continued_fraction_tail(right)
    }
    if positive_continued_fraction_tail(left) and positive_continued_fraction_tail(right) {
        positive_continued_fraction_tail_append(left, right)
        positive_continued_fraction_tail(left + right)
    }
}

/// A positive tail appended to a positive tail gives valid nonempty
/// continued-fraction coefficients.
theorem finite_continued_fraction_coefficients_cons_append_intro(
    head: Nat, left_tail: List[Nat], right_tail: List[Nat]
) {
    positive_continued_fraction_tail(left_tail)
        and positive_continued_fraction_tail(right_tail)
        implies finite_continued_fraction_coefficients(
            List.cons(head, left_tail + right_tail))
} by {
    if positive_continued_fraction_tail(left_tail)
        and positive_continued_fraction_tail(right_tail) {
        positive_continued_fraction_tail_append(left_tail, right_tail)
        finite_continued_fraction_coefficients(List.cons(head, left_tail + right_tail))
    }
}

/// A product of positive natural numbers is positive.
theorem nat_mul_positive(left: Nat, right: Nat) {
    Nat.0 < left and Nat.0 < right implies Nat.0 < left * right
} by {
    if Nat.0 < left and Nat.0 < right {
        if left = Nat.0 {
            lt_not_ref(Nat.0)
            false
        }
        lt_mul_both(left, Nat.0, right)
        mul_zero_right(left)
        Nat.0 < left * right
    }
}

/// Adding any natural to a positive natural gives a positive natural.
theorem nat_add_positive_left(left: Nat, right: Nat) {
    Nat.0 < left implies Nat.0 < left + right
} by {
    if Nat.0 < left {
        if left + right = Nat.0 {
            add_to_zero(left, right)
            lt_not_ref(Nat.0)
            false
        }
        left + right != Nat.0
        pos_of_ne_zero(left + right)
    }
}

/// A positive natural is nonzero.
theorem nat_positive_ne_zero(n: Nat) {
    Nat.0 < n implies n != Nat.0
} by {
    if Nat.0 < n {
        if n = Nat.0 {
            lt_not_ref(Nat.0)
            false
        }
    }
}

/// The next continuant state is positive when the current state and coefficient are positive.
theorem continuant_state_next_positive(previous: Nat, current: Nat, head: Nat) {
    Nat.0 < current and Nat.0 < head implies Nat.0 < current * head + previous
} by {
    if Nat.0 < current and Nat.0 < head {
        nat_mul_positive(current, head)
        Nat.0 < current * head
        nat_add_positive_left(current * head, previous)
    }
}

/// A singleton list of coefficients is a finite simple continued fraction.
theorem finite_continued_fraction_coefficients_singleton(head: Nat) {
    finite_continued_fraction_coefficients(List.cons(head, List.nil[Nat]))
}

/// The continued fraction with one coefficient.
let continued_fraction_singleton(head: Nat) -> cf: ContinuedFraction satisfy {
    ContinuedFraction.new(List.cons(head, List.nil[Nat])) = Option.some(cf)
} by {
    finite_continued_fraction_coefficients_singleton(head)
}

attributes ContinuedFraction {
    /// The continued fraction with one coefficient.
    let singleton = continued_fraction_singleton

    /// The rational value of the continued fraction.
    define value(self) -> Rat {
        continued_fraction_value(self.coefficients)
    }

    /// The continuant associated to the continued fraction.
    define continuant(self) -> Nat {
        continuant(self.coefficients)
    }

    /// The numerator of the finite convergent.
    define numerator(self) -> Nat {
        continued_fraction_numerator(self.coefficients)
    }

    /// The denominator of the finite convergent.
    define denominator(self) -> Nat {
        continued_fraction_denominator(self.coefficients)
    }

    /// The numerator-denominator pair of the finite convergent.
    define convergent(self) -> Pair[Nat, Nat] {
        continued_fraction_convergent(self.coefficients)
    }

    /// The first `n` coefficients.
    define prefix_coefficients(self, n: Nat) -> List[Nat] {
        continued_fraction_prefix_coefficients(self.coefficients, n)
    }

    /// The coefficients left after removing the first `n` coefficients.
    define suffix_coefficients(self, n: Nat) -> List[Nat] {
        continued_fraction_suffix_coefficients(self.coefficients, n)
    }

    /// True when the first `n` coefficients form a finite simple continued
    /// fraction.
    define has_valid_prefix(self, n: Nat) -> Bool {
        finite_continued_fraction_prefix_coefficients(self.coefficients, n)
    }

    /// True when the coefficients after removing the first `n` coefficients
    /// form a finite simple continued fraction.
    define has_valid_suffix(self, n: Nat) -> Bool {
        finite_continued_fraction_suffix_coefficients(self.coefficients, n)
    }

    /// True when the first `n` coefficients have positive tail.
    define has_positive_prefix_tail(self, n: Nat) -> Bool {
        positive_continued_fraction_prefix_tail(self.coefficients, n)
    }

    /// True when the coefficients after removing the first `n` coefficients
    /// have positive tail.
    define has_positive_suffix_tail(self, n: Nat) -> Bool {
        positive_continued_fraction_suffix_tail(self.coefficients, n)
    }

    /// The rational value of the first `n` coefficients.
    define prefix_value(self, n: Nat) -> Rat {
        continued_fraction_prefix_value(self.coefficients, n)
    }

    /// The rational value after removing the first `n` coefficients.
    define suffix_value(self, n: Nat) -> Rat {
        continued_fraction_suffix_value(self.coefficients, n)
    }

    /// The continuant of the first `n` coefficients.
    define prefix_continuant(self, n: Nat) -> Nat {
        continued_fraction_prefix_continuant(self.coefficients, n)
    }

    /// The continuant after removing the first `n` coefficients.
    define suffix_continuant(self, n: Nat) -> Nat {
        continued_fraction_suffix_continuant(self.coefficients, n)
    }

    /// The numerator of the first `n` coefficients.
    define prefix_numerator(self, n: Nat) -> Nat {
        continued_fraction_prefix_numerator(self.coefficients, n)
    }

    /// The numerator after removing the first `n` coefficients.
    define suffix_numerator(self, n: Nat) -> Nat {
        continued_fraction_suffix_numerator(self.coefficients, n)
    }

    /// The denominator of the first `n` coefficients.
    define prefix_denominator(self, n: Nat) -> Nat {
        continued_fraction_prefix_denominator(self.coefficients, n)
    }

    /// The denominator after removing the first `n` coefficients.
    define suffix_denominator(self, n: Nat) -> Nat {
        continued_fraction_suffix_denominator(self.coefficients, n)
    }

    /// The convergent pair of the first `n` coefficients.
    define prefix_convergent(self, n: Nat) -> Pair[Nat, Nat] {
        continued_fraction_prefix_convergent(self.coefficients, n)
    }

    /// The convergent pair after removing the first `n` coefficients.
    define suffix_convergent(self, n: Nat) -> Pair[Nat, Nat] {
        continued_fraction_suffix_convergent(self.coefficients, n)
    }
}

/// A continued fraction has valid coefficients.
theorem continued_fraction_coefficients_valid(cf: ContinuedFraction) {
    finite_continued_fraction_coefficients(cf.coefficients)
}

/// Rebuilding a continued fraction from its coefficients gives the original fraction.
theorem continued_fraction_new_self(cf: ContinuedFraction) {
    ContinuedFraction.new(cf.coefficients) = Option.some(cf)
}

/// The value method is the coefficient-list value.
theorem continued_fraction_value_eq_coefficients_value(cf: ContinuedFraction) {
    cf.value = continued_fraction_value(cf.coefficients)
}

/// The continuant method is the coefficient-list continuant.
theorem continued_fraction_continuant_eq_coefficients_continuant(cf: ContinuedFraction) {
    cf.continuant = continuant(cf.coefficients)
}

/// The numerator method is the coefficient-list numerator.
theorem continued_fraction_numerator_eq_coefficients_numerator(cf: ContinuedFraction) {
    cf.numerator = continued_fraction_numerator(cf.coefficients)
}

/// The denominator method is the coefficient-list denominator.
theorem continued_fraction_denominator_eq_coefficients_denominator(cf: ContinuedFraction) {
    cf.denominator = continued_fraction_denominator(cf.coefficients)
}

/// The convergent method is the coefficient-list convergent.
theorem continued_fraction_convergent_eq_coefficients_convergent(cf: ContinuedFraction) {
    cf.convergent = continued_fraction_convergent(cf.coefficients)
}

/// The prefix-coefficients method is the coefficient-list prefix.
theorem continued_fraction_prefix_coefficients_eq_coefficients_prefix(
    cf: ContinuedFraction, n: Nat
) {
    cf.prefix_coefficients(n) =
        continued_fraction_prefix_coefficients(cf.coefficients, n)
}

/// The suffix-coefficients method is the coefficient-list suffix.
theorem continued_fraction_suffix_coefficients_eq_coefficients_suffix(
    cf: ContinuedFraction, n: Nat
) {
    cf.suffix_coefficients(n) =
        continued_fraction_suffix_coefficients(cf.coefficients, n)
}

/// The prefix-validity method is the coefficient-list prefix-validity
/// predicate.
theorem continued_fraction_has_valid_prefix_eq_coefficients_valid_prefix(
    cf: ContinuedFraction, n: Nat
) {
    cf.has_valid_prefix(n) =
        finite_continued_fraction_prefix_coefficients(cf.coefficients, n)
}

/// The suffix-validity method is the coefficient-list suffix-validity
/// predicate.
theorem continued_fraction_has_valid_suffix_eq_coefficients_valid_suffix(
    cf: ContinuedFraction, n: Nat
) {
    cf.has_valid_suffix(n) =
        finite_continued_fraction_suffix_coefficients(cf.coefficients, n)
}

/// The prefix-tail method is the coefficient-list prefix-tail predicate.
theorem continued_fraction_has_positive_prefix_tail_eq_coefficients_prefix_tail(
    cf: ContinuedFraction, n: Nat
) {
    cf.has_positive_prefix_tail(n) =
        positive_continued_fraction_prefix_tail(cf.coefficients, n)
}

/// The suffix-tail method is the coefficient-list suffix-tail predicate.
theorem continued_fraction_has_positive_suffix_tail_eq_coefficients_suffix_tail(
    cf: ContinuedFraction, n: Nat
) {
    cf.has_positive_suffix_tail(n) =
        positive_continued_fraction_suffix_tail(cf.coefficients, n)
}

/// The prefix-value method is the coefficient-list prefix value.
theorem continued_fraction_prefix_value_eq_coefficients_prefix_value(
    cf: ContinuedFraction, n: Nat
) {
    cf.prefix_value(n) = continued_fraction_prefix_value(cf.coefficients, n)
}

/// The suffix-value method is the coefficient-list suffix value.
theorem continued_fraction_suffix_value_eq_coefficients_suffix_value(
    cf: ContinuedFraction, n: Nat
) {
    cf.suffix_value(n) = continued_fraction_suffix_value(cf.coefficients, n)
}

/// The prefix-continuant method is the coefficient-list prefix continuant.
theorem continued_fraction_prefix_continuant_eq_coefficients_prefix_continuant(
    cf: ContinuedFraction, n: Nat
) {
    cf.prefix_continuant(n) = continued_fraction_prefix_continuant(cf.coefficients, n)
}

/// The suffix-continuant method is the coefficient-list suffix continuant.
theorem continued_fraction_suffix_continuant_eq_coefficients_suffix_continuant(
    cf: ContinuedFraction, n: Nat
) {
    cf.suffix_continuant(n) =
        continued_fraction_suffix_continuant(cf.coefficients, n)
}

/// The prefix-numerator method is the coefficient-list prefix numerator.
theorem continued_fraction_prefix_numerator_eq_coefficients_prefix_numerator(
    cf: ContinuedFraction, n: Nat
) {
    cf.prefix_numerator(n) = continued_fraction_prefix_numerator(cf.coefficients, n)
}

/// The suffix-numerator method is the coefficient-list suffix numerator.
theorem continued_fraction_suffix_numerator_eq_coefficients_suffix_numerator(
    cf: ContinuedFraction, n: Nat
) {
    cf.suffix_numerator(n) =
        continued_fraction_suffix_numerator(cf.coefficients, n)
}

/// The prefix-denominator method is the coefficient-list prefix denominator.
theorem continued_fraction_prefix_denominator_eq_coefficients_prefix_denominator(
    cf: ContinuedFraction, n: Nat
) {
    cf.prefix_denominator(n) =
        continued_fraction_prefix_denominator(cf.coefficients, n)
}

/// The suffix-denominator method is the coefficient-list suffix denominator.
theorem continued_fraction_suffix_denominator_eq_coefficients_suffix_denominator(
    cf: ContinuedFraction, n: Nat
) {
    cf.suffix_denominator(n) =
        continued_fraction_suffix_denominator(cf.coefficients, n)
}

/// The prefix-convergent method is the coefficient-list prefix convergent.
theorem continued_fraction_prefix_convergent_eq_coefficients_prefix_convergent(
    cf: ContinuedFraction, n: Nat
) {
    cf.prefix_convergent(n) = continued_fraction_prefix_convergent(cf.coefficients, n)
}

/// The suffix-convergent method is the coefficient-list suffix convergent.
theorem continued_fraction_suffix_convergent_eq_coefficients_suffix_convergent(
    cf: ContinuedFraction, n: Nat
) {
    cf.suffix_convergent(n) =
        continued_fraction_suffix_convergent(cf.coefficients, n)
}

/// The singleton constructor has the expected coefficient list.
theorem continued_fraction_singleton_coefficients(head: Nat) {
    ContinuedFraction.singleton(head).coefficients = List.cons(head, List.nil[Nat])
} by {
}

/// Coefficient validity unfolds over a nonempty list.
theorem finite_continued_fraction_coefficients_cons(head: Nat, tail: List[Nat]) {
    finite_continued_fraction_coefficients(List.cons(head, tail)) =
        positive_continued_fraction_tail(tail)
}

/// A valid nonempty coefficient list has a positive tail.
theorem finite_continued_fraction_coefficients_cons_tail(head: Nat, tail: List[Nat]) {
    finite_continued_fraction_coefficients(List.cons(head, tail))
        implies positive_continued_fraction_tail(tail)
}

/// A positive tail gives a valid nonempty coefficient list.
theorem finite_continued_fraction_coefficients_cons_intro(head: Nat, tail: List[Nat]) {
    positive_continued_fraction_tail(tail)
        implies finite_continued_fraction_coefficients(List.cons(head, tail))
}

/// A two-term list with a positive second coefficient is valid.
theorem finite_continued_fraction_coefficients_pair_intro(head: Nat, next: Nat) {
    Nat.0 < next implies
        finite_continued_fraction_coefficients(List.cons(head, List.cons(next, List.nil[Nat])))
} by {
    if Nat.0 < next {
        positive_continued_fraction_tail_nil
        positive_continued_fraction_tail_cons_intro(next, List.nil[Nat])
        finite_continued_fraction_coefficients_cons(head, List.cons(next, List.nil[Nat]))
    }
}

/// A valid two-term list has a positive second coefficient.
theorem finite_continued_fraction_coefficients_pair_tail_positive(head: Nat, next: Nat) {
    finite_continued_fraction_coefficients(List.cons(head, List.cons(next, List.nil[Nat])))
        implies Nat.0 < next
} by {
    if finite_continued_fraction_coefficients(List.cons(head, List.cons(next, List.nil[Nat]))) {
        finite_continued_fraction_coefficients_cons_tail(head, List.cons(next, List.nil[Nat]))
        positive_continued_fraction_tail_cons_head(next, List.nil[Nat])
    }
}

/// A three-term list with positive tail coefficients is valid.
theorem finite_continued_fraction_coefficients_triple_intro(head: Nat, next: Nat,
    third: Nat) {
    Nat.0 < next and Nat.0 < third implies
        finite_continued_fraction_coefficients(
            List.cons(head, List.cons(next, List.cons(third, List.nil[Nat]))))
} by {
    if Nat.0 < next and Nat.0 < third {
        positive_continued_fraction_tail_pair_intro(next, third)
        finite_continued_fraction_coefficients_cons_intro(head,
            List.cons(next, List.cons(third, List.nil[Nat])))
    }
}

/// A valid three-term coefficient list has a positive second coefficient.
theorem finite_continued_fraction_coefficients_triple_second_positive(head: Nat,
    next: Nat, third: Nat) {
    finite_continued_fraction_coefficients(
        List.cons(head, List.cons(next, List.cons(third, List.nil[Nat]))))
        implies Nat.0 < next
} by {
    if finite_continued_fraction_coefficients(
        List.cons(head, List.cons(next, List.cons(third, List.nil[Nat])))) {
        finite_continued_fraction_coefficients_cons_tail(head,
            List.cons(next, List.cons(third, List.nil[Nat])))
        positive_continued_fraction_tail(List.cons(next, List.cons(third, List.nil[Nat])))
        positive_continued_fraction_tail_cons_head(next, List.cons(third, List.nil[Nat]))
    }
}

/// A valid three-term coefficient list has a positive third coefficient.
theorem finite_continued_fraction_coefficients_triple_third_positive(head: Nat,
    next: Nat, third: Nat) {
    finite_continued_fraction_coefficients(
        List.cons(head, List.cons(next, List.cons(third, List.nil[Nat]))))
        implies Nat.0 < third
} by {
    if finite_continued_fraction_coefficients(
        List.cons(head, List.cons(next, List.cons(third, List.nil[Nat])))) {
        finite_continued_fraction_coefficients_cons_tail(head,
            List.cons(next, List.cons(third, List.nil[Nat])))
        positive_continued_fraction_tail_cons_tail(next, List.cons(third, List.nil[Nat]))
        positive_continued_fraction_tail_singleton_positive(third)
        Nat.0 < third
    }
}

/// The empty coefficient list has value zero.
theorem continued_fraction_value_nil {
    continued_fraction_value(List.nil[Nat]) = Rat.0
}

/// The value of a nonempty list unfolds by adding the reciprocal of the tail.
theorem continued_fraction_value_cons(head: Nat, tail: List[Nat]) {
    continued_fraction_value(List.cons(head, tail)) =
        Rat.from_nat(head) + continued_fraction_value(tail).inverse
}

/// A singleton continued fraction has the value of its single coefficient.
theorem continued_fraction_value_singleton(head: Nat) {
    continued_fraction_value(List.cons(head, List.nil[Nat])) = Rat.from_nat(head)
} by {
    continued_fraction_value_nil
    zero_recip(Rat.0)
    add_zero_right(Rat.from_nat(head))
}

/// A singleton continued fraction has the value of its single coefficient.
theorem continued_fraction_singleton_value(head: Nat) {
    ContinuedFraction.singleton(head).value = Rat.from_nat(head)
} by {
    continued_fraction_singleton_coefficients(head)
    continued_fraction_value_eq_coefficients_value(ContinuedFraction.singleton(head))
    continued_fraction_value_singleton(head)
}

/// The continuant of the empty list is one.
theorem continuant_nil {
    continuant(List.nil[Nat]) = Nat.1
}

/// The empty state returns the current continuant.
theorem continuant_state_nil(previous: Nat, current: Nat) {
    continuant_state(List.nil[Nat], previous, current) = current
}

/// A nonempty state advances the continuant recurrence by one coefficient.
theorem continuant_state_cons(head: Nat, tail: List[Nat], previous: Nat, current: Nat) {
    continuant_state(List.cons(head, tail), previous, current) =
        continuant_state(tail, current, current * head + previous)
}

/// The continuant of a singleton list is its single coefficient.
theorem continuant_singleton(head: Nat) {
    continuant(List.cons(head, List.nil[Nat])) = head
} by {
    continuant_state(List.cons(head, List.nil[Nat]), Nat.0, Nat.1) =
        continuant_state(List.nil[Nat], Nat.1, Nat.1 * head + Nat.0)
    mul_one_left(head)
}

/// A singleton continued fraction has continuant equal to its coefficient.
theorem continued_fraction_singleton_continuant(head: Nat) {
    ContinuedFraction.singleton(head).continuant = head
} by {
    continued_fraction_singleton_coefficients(head)
    continued_fraction_continuant_eq_coefficients_continuant(ContinuedFraction.singleton(head))
    continuant_singleton(head)
}

/// The continuant of a two-element list is `head * next + 1`.
theorem continuant_pair(head: Nat, next: Nat) {
    continuant(List.cons(head, List.cons(next, List.nil[Nat]))) =
        head * next + Nat.1
} by {
    continuant_state(List.cons(next, List.nil[Nat]), Nat.1, Nat.1 * head + Nat.0) =
        continuant_state(List.nil[Nat], Nat.1 * head + Nat.0,
            (Nat.1 * head + Nat.0) * next + Nat.1)
    mul_one_left(head)
}

/// The continuant of a three-element list follows the third recurrence step.
theorem continuant_triple(head: Nat, next: Nat, third: Nat) {
    continuant(List.cons(head, List.cons(next, List.cons(third, List.nil[Nat])))) =
        (head * next + Nat.1) * third + head
} by {
    continuant_state(List.cons(next, List.cons(third, List.nil[Nat])), Nat.1,
        Nat.1 * head + Nat.0) =
        continuant_state(List.cons(third, List.nil[Nat]), Nat.1 * head + Nat.0,
            (Nat.1 * head + Nat.0) * next + Nat.1)
    continuant_state(List.nil[Nat], (Nat.1 * head + Nat.0) * next + Nat.1,
        ((Nat.1 * head + Nat.0) * next + Nat.1) * third +
            (Nat.1 * head + Nat.0)) =
        ((Nat.1 * head + Nat.0) * next + Nat.1) * third + (Nat.1 * head + Nat.0)
    mul_one_left(head)
}

/// A singleton continuant state advances once.
theorem continuant_state_singleton(head: Nat, previous: Nat, current: Nat) {
    continuant_state(List.cons(head, List.nil[Nat]), previous, current) =
        current * head + previous
} by {
    continuant_state_cons(head, List.nil[Nat], previous, current)
    continuant_state_nil(current, current * head + previous)
}

/// A two-coefficient continuant state advances twice.
theorem continuant_state_pair(head: Nat, next: Nat, previous: Nat, current: Nat) {
    continuant_state(List.cons(head, List.cons(next, List.nil[Nat])),
        previous, current) =
        (current * head + previous) * next + current
} by {
    continuant_state_cons(head, List.cons(next, List.nil[Nat]), previous, current)
    continuant_state_singleton(next, current, current * head + previous)
}

/// A three-coefficient continuant state advances three times.
theorem continuant_state_triple(head: Nat, next: Nat, third: Nat,
    previous: Nat, current: Nat) {
    continuant_state(List.cons(head, List.cons(next, List.cons(third, List.nil[Nat]))),
        previous, current) =
        ((current * head + previous) * next + current) * third +
            (current * head + previous)
} by {
    continuant_state_cons(head, List.cons(next, List.cons(third, List.nil[Nat])),
        previous, current)
    continuant_state_pair(next, third, current, current * head + previous)
}

/// The empty continued fraction has numerator zero.
theorem continued_fraction_numerator_nil {
    continued_fraction_numerator(List.nil[Nat]) = Nat.0
}

/// A singleton continued fraction has numerator equal to its coefficient.
theorem continued_fraction_numerator_singleton(head: Nat) {
    continued_fraction_numerator(List.cons(head, List.nil[Nat])) = head
}

/// A singleton continued fraction has numerator equal to its coefficient.
theorem continued_fraction_singleton_numerator(head: Nat) {
    ContinuedFraction.singleton(head).numerator = head
} by {
    continued_fraction_singleton_coefficients(head)
    continued_fraction_numerator_eq_coefficients_numerator(ContinuedFraction.singleton(head))
    continued_fraction_numerator_singleton(head)
}

/// A two-term continued fraction has numerator `head * next + 1`.
theorem continued_fraction_numerator_pair(head: Nat, next: Nat) {
    continued_fraction_numerator(List.cons(head, List.cons(next, List.nil[Nat]))) =
        head * next + Nat.1
}

/// A three-term continued fraction has numerator given by the third continuant.
theorem continued_fraction_numerator_triple(head: Nat, next: Nat, third: Nat) {
    continued_fraction_numerator(
        List.cons(head, List.cons(next, List.cons(third, List.nil[Nat])))) =
        (head * next + Nat.1) * third + head
} by {
    continuant_triple(head, next, third)
}

/// The empty continued fraction has denominator one.
theorem continued_fraction_denominator_nil {
    continued_fraction_denominator(List.nil[Nat]) = Nat.1
}

/// The empty convergent is the conventional pair `0/1`.
theorem continued_fraction_convergent_nil {
    continued_fraction_convergent(List.nil[Nat]) = Pair.new(Nat.0, Nat.1)
}

/// Taking zero coefficients gives the empty prefix.
theorem continued_fraction_prefix_coefficients_zero(coefficients: List[Nat]) {
    continued_fraction_prefix_coefficients(coefficients, Nat.0) = List.nil[Nat]
} by {
    list_take_zero(coefficients)
}

/// Every prefix of the empty coefficient list is empty.
theorem continued_fraction_prefix_coefficients_nil(n: Nat) {
    continued_fraction_prefix_coefficients(List.nil[Nat], n) = List.nil[Nat]
} by {
    list_take_nil[Nat](n)
}

/// A successor prefix of a nonempty coefficient list keeps the head and takes
/// a predecessor prefix of the tail.
theorem continued_fraction_prefix_coefficients_cons_suc(
    head: Nat, tail: List[Nat], n: Nat
) {
    continued_fraction_prefix_coefficients(List.cons(head, tail), n.suc) =
        List.cons(head, continued_fraction_prefix_coefficients(tail, n))
} by {
    list_take_cons_suc(head, tail, n)
}

/// The whole-length prefix is the original coefficient list.
theorem continued_fraction_prefix_coefficients_length(coefficients: List[Nat]) {
    continued_fraction_prefix_coefficients(coefficients, coefficients.length) =
        coefficients
} by {
    list_take_length(coefficients)
}

/// Prefix and suffix coefficients reassemble the original coefficient list.
theorem continued_fraction_prefix_append_suffix(
    coefficients: List[Nat], n: Nat
) {
    continued_fraction_prefix_coefficients(coefficients, n) +
        continued_fraction_suffix_coefficients(coefficients, n) = coefficients
} by {
    list_take_append_drop(coefficients, n)
}

/// The whole coefficient list is the length-prefix of a continued fraction.
theorem continued_fraction_prefix_coefficients_full(cf: ContinuedFraction) {
    cf.prefix_coefficients(cf.coefficients.length) = cf.coefficients
} by {
    continued_fraction_prefix_coefficients_eq_coefficients_prefix(
        cf, cf.coefficients.length)
    continued_fraction_prefix_coefficients_length(cf.coefficients)
}

/// A continued fraction's prefix and suffix coefficients reassemble its
/// coefficient list.
theorem continued_fraction_prefix_append_suffix_coefficients(
    cf: ContinuedFraction, n: Nat
) {
    cf.prefix_coefficients(n) + cf.suffix_coefficients(n) = cf.coefficients
} by {
    continued_fraction_prefix_coefficients_eq_coefficients_prefix(cf, n)
    continued_fraction_suffix_coefficients_eq_coefficients_suffix(cf, n)
    continued_fraction_prefix_append_suffix(cf.coefficients, n)
}

/// The zero prefix is not a finite simple continued fraction.
theorem finite_continued_fraction_prefix_coefficients_zero(
    coefficients: List[Nat]
) {
    not finite_continued_fraction_prefix_coefficients(coefficients, Nat.0)
} by {
    continued_fraction_prefix_coefficients_zero(coefficients)
}

/// Every zero prefix has positive tail.
theorem positive_continued_fraction_prefix_tail_zero(coefficients: List[Nat]) {
    positive_continued_fraction_prefix_tail(coefficients, Nat.0)
} by {
    continued_fraction_prefix_coefficients_zero(coefficients)
    positive_continued_fraction_tail_nil
}

/// The zero prefix has value zero.
theorem continued_fraction_prefix_value_zero(coefficients: List[Nat]) {
    continued_fraction_prefix_value(coefficients, Nat.0) = Rat.0
} by {
    continued_fraction_prefix_coefficients_zero(coefficients)
    continued_fraction_value_nil
}

/// Every prefix of the empty coefficient list has value zero.
theorem continued_fraction_prefix_value_nil(n: Nat) {
    continued_fraction_prefix_value(List.nil[Nat], n) = Rat.0
} by {
    continued_fraction_prefix_coefficients_nil(n)
    continued_fraction_value_nil
}

/// The zero prefix has continuant one.
theorem continued_fraction_prefix_continuant_zero(coefficients: List[Nat]) {
    continued_fraction_prefix_continuant(coefficients, Nat.0) = Nat.1
} by {
    continued_fraction_prefix_coefficients_zero(coefficients)
    continuant_nil
}

/// Every prefix of the empty coefficient list has continuant one.
theorem continued_fraction_prefix_continuant_nil(n: Nat) {
    continued_fraction_prefix_continuant(List.nil[Nat], n) = Nat.1
} by {
    continued_fraction_prefix_coefficients_nil(n)
    continuant_nil
}

/// The zero prefix has numerator zero.
theorem continued_fraction_prefix_numerator_zero(coefficients: List[Nat]) {
    continued_fraction_prefix_numerator(coefficients, Nat.0) = Nat.0
} by {
    continued_fraction_prefix_coefficients_zero(coefficients)
    continued_fraction_numerator_nil
}

/// Every prefix of the empty coefficient list has numerator zero.
theorem continued_fraction_prefix_numerator_nil(n: Nat) {
    continued_fraction_prefix_numerator(List.nil[Nat], n) = Nat.0
} by {
    continued_fraction_prefix_coefficients_nil(n)
    continued_fraction_numerator_nil
}

/// The zero prefix has denominator one.
theorem continued_fraction_prefix_denominator_zero(coefficients: List[Nat]) {
    continued_fraction_prefix_denominator(coefficients, Nat.0) = Nat.1
} by {
    continued_fraction_prefix_coefficients_zero(coefficients)
    continued_fraction_denominator_nil
}

/// Every prefix of the empty coefficient list has denominator one.
theorem continued_fraction_prefix_denominator_nil(n: Nat) {
    continued_fraction_prefix_denominator(List.nil[Nat], n) = Nat.1
} by {
    continued_fraction_prefix_coefficients_nil(n)
    continued_fraction_denominator_nil
}

/// The zero prefix has the conventional empty convergent.
theorem continued_fraction_prefix_convergent_zero(coefficients: List[Nat]) {
    continued_fraction_prefix_convergent(coefficients, Nat.0) =
        Pair.new(Nat.0, Nat.1)
} by {
    continued_fraction_prefix_coefficients_zero(coefficients)
    continued_fraction_convergent_nil
}

/// Every prefix of the empty coefficient list has the conventional empty
/// convergent.
theorem continued_fraction_prefix_convergent_nil(n: Nat) {
    continued_fraction_prefix_convergent(List.nil[Nat], n) = Pair.new(Nat.0, Nat.1)
} by {
    continued_fraction_prefix_coefficients_nil(n)
    continued_fraction_convergent_nil
}

/// The empty convergent has numerator zero.
theorem continued_fraction_convergent_nil_first {
    continued_fraction_convergent(List.nil[Nat]).first = Nat.0
} by {
    continued_fraction_convergent_nil
    pair_new_first(Nat.0, Nat.1)
}

/// The empty convergent has denominator one.
theorem continued_fraction_convergent_nil_second {
    continued_fraction_convergent(List.nil[Nat]).second = Nat.1
} by {
    continued_fraction_convergent_nil
    pair_new_second(Nat.0, Nat.1)
}

/// The empty convergent has positive denominator.
theorem continued_fraction_convergent_nil_second_positive {
    Nat.0 < continued_fraction_convergent(List.nil[Nat]).second
} by {
    continued_fraction_convergent_nil_second
    Nat.0 < Nat.1
}

/// The empty convergent first projection matches the numerator definition.
theorem continued_fraction_convergent_nil_first_eq_numerator {
    continued_fraction_convergent(List.nil[Nat]).first =
        continued_fraction_numerator(List.nil[Nat])
} by {
    continued_fraction_convergent_nil_first
    continued_fraction_numerator_nil
}

/// The empty convergent second projection matches the denominator definition.
theorem continued_fraction_convergent_nil_second_eq_denominator {
    continued_fraction_convergent(List.nil[Nat]).second =
        continued_fraction_denominator(List.nil[Nat])
} by {
    continued_fraction_convergent_nil_second
    continued_fraction_denominator_nil
}

/// The empty convergent state returns the current numerator and denominator.
theorem continued_fraction_convergent_state_nil(previous_numerator: Nat,
    current_numerator: Nat, previous_denominator: Nat, current_denominator: Nat) {
    continued_fraction_convergent_state(List.nil[Nat], previous_numerator,
        current_numerator, previous_denominator, current_denominator) =
        Pair.new(current_numerator, current_denominator)
}

/// The first projection of an empty convergent state is the current numerator.
theorem continued_fraction_convergent_state_nil_first(previous_numerator: Nat,
    current_numerator: Nat, previous_denominator: Nat, current_denominator: Nat) {
    continued_fraction_convergent_state(List.nil[Nat], previous_numerator,
        current_numerator, previous_denominator, current_denominator).first =
        current_numerator
} by {
    continued_fraction_convergent_state_nil(previous_numerator, current_numerator,
        previous_denominator, current_denominator)
    pair_new_first(current_numerator, current_denominator)
}

/// The second projection of an empty convergent state is the current denominator.
theorem continued_fraction_convergent_state_nil_second(previous_numerator: Nat,
    current_numerator: Nat, previous_denominator: Nat, current_denominator: Nat) {
    continued_fraction_convergent_state(List.nil[Nat], previous_numerator,
        current_numerator, previous_denominator, current_denominator).second =
        current_denominator
} by {
    continued_fraction_convergent_state_nil(previous_numerator, current_numerator,
        previous_denominator, current_denominator)
    pair_new_second(current_numerator, current_denominator)
}

/// A nonempty convergent state advances both numerator and denominator recurrences.
theorem continued_fraction_convergent_state_cons(head: Nat, tail: List[Nat],
    previous_numerator: Nat, current_numerator: Nat,
    previous_denominator: Nat, current_denominator: Nat) {
    continued_fraction_convergent_state(List.cons(head, tail), previous_numerator,
        current_numerator, previous_denominator, current_denominator) =
        continued_fraction_convergent_state(tail,
            current_numerator,
            current_numerator * head + previous_numerator,
            current_denominator,
            current_denominator * head + previous_denominator)
}

/// The first projection of a nonempty state follows the numerator recurrence.
theorem continued_fraction_convergent_state_cons_first(head: Nat, tail: List[Nat],
    previous_numerator: Nat, current_numerator: Nat,
    previous_denominator: Nat, current_denominator: Nat) {
    continued_fraction_convergent_state(List.cons(head, tail), previous_numerator,
        current_numerator, previous_denominator, current_denominator).first =
        continued_fraction_convergent_state(tail,
            current_numerator,
            current_numerator * head + previous_numerator,
            current_denominator,
            current_denominator * head + previous_denominator).first
} by {
    continued_fraction_convergent_state_cons(head, tail, previous_numerator,
        current_numerator, previous_denominator, current_denominator)
}

/// The second projection of a nonempty state follows the denominator recurrence.
theorem continued_fraction_convergent_state_cons_second(head: Nat, tail: List[Nat],
    previous_numerator: Nat, current_numerator: Nat,
    previous_denominator: Nat, current_denominator: Nat) {
    continued_fraction_convergent_state(List.cons(head, tail), previous_numerator,
        current_numerator, previous_denominator, current_denominator).second =
        continued_fraction_convergent_state(tail,
            current_numerator,
            current_numerator * head + previous_numerator,
            current_denominator,
            current_denominator * head + previous_denominator).second
} by {
    continued_fraction_convergent_state_cons(head, tail, previous_numerator,
        current_numerator, previous_denominator, current_denominator)
}

/// True when the first projection of a convergent state is its numerator
/// continuant state.
define continued_fraction_convergent_state_first_projection_at(
    coefficients: List[Nat], previous_numerator: Nat, current_numerator: Nat,
    previous_denominator: Nat, current_denominator: Nat
) -> Bool {
    continued_fraction_convergent_state(coefficients, previous_numerator,
        current_numerator, previous_denominator, current_denominator).first =
        continuant_state(coefficients, previous_numerator, current_numerator)
}

/// True when the second projection of a convergent state is its denominator
/// continuant state.
define continued_fraction_convergent_state_second_projection_at(
    coefficients: List[Nat], previous_numerator: Nat, current_numerator: Nat,
    previous_denominator: Nat, current_denominator: Nat
) -> Bool {
    continued_fraction_convergent_state(coefficients, previous_numerator,
        current_numerator, previous_denominator, current_denominator).second =
        continuant_state(coefficients, previous_denominator, current_denominator)
}

/// The empty convergent state has its first projection equal to the numerator
/// continuant state.
theorem continued_fraction_convergent_state_first_projection_at_nil(
    pn: Nat, cn: Nat, pd: Nat, cd: Nat
) {
    continued_fraction_convergent_state_first_projection_at(List.nil[Nat],
        pn, cn, pd, cd)
} by {
    continued_fraction_convergent_state_nil_first(pn, cn, pd, cd)
    continuant_state_nil(pn, cn)
}

/// The first-projection continuant-state property advances over one
/// coefficient.
theorem continued_fraction_convergent_state_first_projection_at_cons(head: Nat,
    tail: List[Nat], pn: Nat, cn: Nat, pd: Nat, cd: Nat) {
    continued_fraction_convergent_state_first_projection_at(tail, cn,
        cn * head + pn, cd, cd * head + pd)
        implies continued_fraction_convergent_state_first_projection_at(
            List.cons(head, tail), pn, cn, pd, cd)
} by {
    if continued_fraction_convergent_state_first_projection_at(tail, cn,
        cn * head + pn, cd, cd * head + pd) {
        continued_fraction_convergent_state_cons_first(head, tail, pn, cn, pd, cd)
        continued_fraction_convergent_state(tail, cn, cn * head + pn,
            cd, cd * head + pd).first =
            continuant_state(tail, cn, cn * head + pn)
        continuant_state_cons(head, tail, pn, cn)
        continued_fraction_convergent_state_first_projection_at(List.cons(head, tail),
            pn, cn, pd, cd)
    }
}

/// The empty convergent state has its second projection equal to the denominator
/// continuant state.
theorem continued_fraction_convergent_state_second_projection_at_nil(
    pn: Nat, cn: Nat, pd: Nat, cd: Nat
) {
    continued_fraction_convergent_state_second_projection_at(List.nil[Nat],
        pn, cn, pd, cd)
} by {
    continued_fraction_convergent_state_nil_second(pn, cn, pd, cd)
    continuant_state_nil(pd, cd)
}

/// The second-projection continuant-state property advances over one
/// coefficient.
theorem continued_fraction_convergent_state_second_projection_at_cons(head: Nat,
    tail: List[Nat], pn: Nat, cn: Nat, pd: Nat, cd: Nat) {
    continued_fraction_convergent_state_second_projection_at(tail, cn,
        cn * head + pn, cd, cd * head + pd)
        implies continued_fraction_convergent_state_second_projection_at(
            List.cons(head, tail), pn, cn, pd, cd)
} by {
    if continued_fraction_convergent_state_second_projection_at(tail, cn,
        cn * head + pn, cd, cd * head + pd) {
        continued_fraction_convergent_state_cons_second(head, tail, pn, cn, pd, cd)
        continued_fraction_convergent_state(tail, cn, cn * head + pn,
            cd, cd * head + pd).second =
            continuant_state(tail, cd, cd * head + pd)
        continuant_state_cons(head, tail, pd, cd)
        continued_fraction_convergent_state_second_projection_at(List.cons(head, tail),
            pn, cn, pd, cd)
    }
}

/// A first-projection predicate gives the underlying equality.
theorem continued_fraction_convergent_state_first_eq_of_projection_at(
    coefficients: List[Nat], pn: Nat, cn: Nat, pd: Nat, cd: Nat
) {
    continued_fraction_convergent_state_first_projection_at(coefficients, pn, cn, pd, cd)
        implies continued_fraction_convergent_state(coefficients, pn, cn, pd, cd).first =
            continuant_state(coefficients, pn, cn)
} by {
    if continued_fraction_convergent_state_first_projection_at(coefficients, pn, cn, pd, cd) {
        continued_fraction_convergent_state(coefficients, pn, cn, pd, cd).first =
            continuant_state(coefficients, pn, cn)
    }
}

/// A first-projection equality gives the first-projection predicate.
theorem continued_fraction_convergent_state_first_projection_at_intro(
    coefficients: List[Nat], pn: Nat, cn: Nat, pd: Nat, cd: Nat
) {
    continued_fraction_convergent_state(coefficients, pn, cn, pd, cd).first =
        continuant_state(coefficients, pn, cn)
        implies continued_fraction_convergent_state_first_projection_at(
            coefficients, pn, cn, pd, cd)
} by {
    if continued_fraction_convergent_state(coefficients, pn, cn, pd, cd).first =
        continuant_state(coefficients, pn, cn) {
        continued_fraction_convergent_state_first_projection_at(
            coefficients, pn, cn, pd, cd)
    }
}

/// A second-projection predicate gives the underlying equality.
theorem continued_fraction_convergent_state_second_eq_of_projection_at(
    coefficients: List[Nat], pn: Nat, cn: Nat, pd: Nat, cd: Nat
) {
    continued_fraction_convergent_state_second_projection_at(coefficients, pn, cn, pd, cd)
        implies continued_fraction_convergent_state(coefficients, pn, cn, pd, cd).second =
            continuant_state(coefficients, pd, cd)
} by {
    if continued_fraction_convergent_state_second_projection_at(coefficients, pn, cn, pd, cd) {
        continued_fraction_convergent_state(coefficients, pn, cn, pd, cd).second =
            continuant_state(coefficients, pd, cd)
    }
}

/// A second-projection equality gives the second-projection predicate.
theorem continued_fraction_convergent_state_second_projection_at_intro(
    coefficients: List[Nat], pn: Nat, cn: Nat, pd: Nat, cd: Nat
) {
    continued_fraction_convergent_state(coefficients, pn, cn, pd, cd).second =
        continuant_state(coefficients, pd, cd)
        implies continued_fraction_convergent_state_second_projection_at(
            coefficients, pn, cn, pd, cd)
} by {
    if continued_fraction_convergent_state(coefficients, pn, cn, pd, cd).second =
        continuant_state(coefficients, pd, cd) {
        continued_fraction_convergent_state_second_projection_at(
            coefficients, pn, cn, pd, cd)
    }
}

/// True if all current denominator states satisfy the first-projection
/// continuant-state property.
define continued_fraction_convergent_state_first_projection_for_current_denominators(
    coefficients: List[Nat], pn: Nat, cn: Nat, pd: Nat
) -> Bool {
    forall(cd: Nat) {
        continued_fraction_convergent_state_first_projection_at(
            coefficients, pn, cn, pd, cd)
    }
}

/// True if all denominator states satisfy the first-projection
/// continuant-state property.
define continued_fraction_convergent_state_first_projection_for_denominators(
    coefficients: List[Nat], pn: Nat, cn: Nat
) -> Bool {
    forall(pd: Nat) {
        continued_fraction_convergent_state_first_projection_for_current_denominators(
            coefficients, pn, cn, pd)
    }
}

/// True if all current numerator states satisfy the first-projection
/// continuant-state property.
define continued_fraction_convergent_state_first_projection_for_current_numerators(
    coefficients: List[Nat], pn: Nat
) -> Bool {
    forall(cn: Nat) {
        continued_fraction_convergent_state_first_projection_for_denominators(
            coefficients, pn, cn)
    }
}

/// True if every numerator-denominator state satisfies the first-projection
/// continuant-state property.
define continued_fraction_convergent_state_first_projection_for_states(
    coefficients: List[Nat]
) -> Bool {
    forall(pn: Nat) {
        continued_fraction_convergent_state_first_projection_for_current_numerators(
            coefficients, pn)
    }
}

/// A current-denominator package gives the corresponding first-projection
/// predicate.
theorem continued_fraction_convergent_state_first_projection_for_current_denominators_at(
    coefficients: List[Nat], pn: Nat, cn: Nat, pd: Nat, cd: Nat
) {
    continued_fraction_convergent_state_first_projection_for_current_denominators(
        coefficients, pn, cn, pd)
        implies continued_fraction_convergent_state_first_projection_at(
            coefficients, pn, cn, pd, cd)
} by {
    if continued_fraction_convergent_state_first_projection_for_current_denominators(
        coefficients, pn, cn, pd) {
        continued_fraction_convergent_state_first_projection_for_current_denominators(
            coefficients, pn, cn, pd) =
            forall(current_denominator: Nat) {
                continued_fraction_convergent_state_first_projection_at(
                    coefficients, pn, cn, pd, current_denominator)
            }
        let h: Bool = continued_fraction_convergent_state_first_projection_at(
            coefficients, pn, cn, pd, cd)
        continued_fraction_convergent_state_first_projection_at(
            coefficients, pn, cn, pd, cd)
    }
}

/// A denominator package gives the corresponding current-denominator package.
theorem continued_fraction_convergent_state_first_projection_for_denominators_at(
    coefficients: List[Nat], pn: Nat, cn: Nat, pd: Nat
) {
    continued_fraction_convergent_state_first_projection_for_denominators(
        coefficients, pn, cn)
        implies
        continued_fraction_convergent_state_first_projection_for_current_denominators(
            coefficients, pn, cn, pd)
} by {
    if continued_fraction_convergent_state_first_projection_for_denominators(
        coefficients, pn, cn) {
        continued_fraction_convergent_state_first_projection_for_denominators(
            coefficients, pn, cn) =
            forall(previous_denominator: Nat) {
                continued_fraction_convergent_state_first_projection_for_current_denominators(
                    coefficients, pn, cn, previous_denominator)
            }
        let h: Bool =
            continued_fraction_convergent_state_first_projection_for_current_denominators(
                coefficients, pn, cn, pd)
        continued_fraction_convergent_state_first_projection_for_current_denominators(
            coefficients, pn, cn, pd)
    }
}

/// A current-numerator package gives the corresponding denominator package.
theorem continued_fraction_convergent_state_first_projection_for_current_numerators_at(
    coefficients: List[Nat], pn: Nat, cn: Nat
) {
    continued_fraction_convergent_state_first_projection_for_current_numerators(
        coefficients, pn)
        implies continued_fraction_convergent_state_first_projection_for_denominators(
            coefficients, pn, cn)
} by {
    if continued_fraction_convergent_state_first_projection_for_current_numerators(
        coefficients, pn) {
        continued_fraction_convergent_state_first_projection_for_current_numerators(
            coefficients, pn) =
            forall(current_numerator: Nat) {
                continued_fraction_convergent_state_first_projection_for_denominators(
                    coefficients, pn, current_numerator)
            }
        let h: Bool =
            continued_fraction_convergent_state_first_projection_for_denominators(
                coefficients, pn, cn)
        continued_fraction_convergent_state_first_projection_for_denominators(
            coefficients, pn, cn)
    }
}

/// A state package gives the corresponding current-numerator package.
theorem continued_fraction_convergent_state_first_projection_for_states_at_numerator(
    coefficients: List[Nat], pn: Nat
) {
    continued_fraction_convergent_state_first_projection_for_states(coefficients)
        implies
        continued_fraction_convergent_state_first_projection_for_current_numerators(
            coefficients, pn)
} by {
    if continued_fraction_convergent_state_first_projection_for_states(coefficients) {
        continued_fraction_convergent_state_first_projection_for_states(coefficients) =
            forall(previous_numerator: Nat) {
                continued_fraction_convergent_state_first_projection_for_current_numerators(
                    coefficients, previous_numerator)
            }
        let h: Bool =
            continued_fraction_convergent_state_first_projection_for_current_numerators(
                coefficients, pn)
        continued_fraction_convergent_state_first_projection_for_current_numerators(
            coefficients, pn)
    }
}

/// A state package gives the corresponding first-projection predicate.
theorem continued_fraction_convergent_state_first_projection_for_states_at(
    coefficients: List[Nat], pn: Nat, cn: Nat, pd: Nat, cd: Nat
) {
    continued_fraction_convergent_state_first_projection_for_states(coefficients)
        implies continued_fraction_convergent_state_first_projection_at(
            coefficients, pn, cn, pd, cd)
} by {
    if continued_fraction_convergent_state_first_projection_for_states(coefficients) {
        continued_fraction_convergent_state_first_projection_for_states_at_numerator(
            coefficients, pn)
        continued_fraction_convergent_state_first_projection_for_current_numerators_at(
            coefficients, pn, cn)
        continued_fraction_convergent_state_first_projection_for_denominators_at(
            coefficients, pn, cn, pd)
        continued_fraction_convergent_state_first_projection_for_current_denominators_at(
            coefficients, pn, cn, pd, cd)
        continued_fraction_convergent_state_first_projection_at(
            coefficients, pn, cn, pd, cd)
    }
}

/// The empty coefficient list satisfies the current-denominator
/// first-projection package.
theorem continued_fraction_convergent_state_first_projection_for_current_denominators_nil(
    pn: Nat, cn: Nat, pd: Nat
) {
    continued_fraction_convergent_state_first_projection_for_current_denominators(
        List.nil[Nat], pn, cn, pd)
} by {
    forall(cd: Nat) {
        continued_fraction_convergent_state_first_projection_at_nil(pn, cn, pd, cd)
    }
}

/// The empty coefficient list satisfies the denominator first-projection
/// package.
theorem continued_fraction_convergent_state_first_projection_for_denominators_nil(
    pn: Nat, cn: Nat
) {
    continued_fraction_convergent_state_first_projection_for_denominators(
        List.nil[Nat], pn, cn)
} by {
    forall(pd: Nat) {
        continued_fraction_convergent_state_first_projection_for_current_denominators_nil(
            pn, cn, pd)
    }
}

/// The empty coefficient list satisfies the current-numerator
/// first-projection package.
theorem continued_fraction_convergent_state_first_projection_for_current_numerators_nil(
    pn: Nat
) {
    continued_fraction_convergent_state_first_projection_for_current_numerators(
        List.nil[Nat], pn)
} by {
    forall(cn: Nat) {
        continued_fraction_convergent_state_first_projection_for_denominators_nil(
            pn, cn)
    }
}

/// The empty coefficient list satisfies the state first-projection package.
theorem continued_fraction_convergent_state_first_projection_for_states_nil {
    continued_fraction_convergent_state_first_projection_for_states(List.nil[Nat])
} by {
    forall(pn: Nat) {
        continued_fraction_convergent_state_first_projection_for_current_numerators_nil(pn)
    }
}

/// The current-denominator first-projection package advances over one
/// coefficient.
theorem continued_fraction_convergent_state_first_projection_for_current_denominators_cons(
    head: Nat, tail: List[Nat], pn: Nat, cn: Nat, pd: Nat
) {
    continued_fraction_convergent_state_first_projection_for_states(tail)
        implies
        continued_fraction_convergent_state_first_projection_for_current_denominators(
            List.cons(head, tail), pn, cn, pd)
} by {
    if continued_fraction_convergent_state_first_projection_for_states(tail) {
        forall(cd: Nat) {
            continued_fraction_convergent_state_first_projection_for_states_at(
                tail, cn, cn * head + pn, cd, cd * head + pd)
            continued_fraction_convergent_state_first_projection_at_cons(
                head, tail, pn, cn, pd, cd)
            continued_fraction_convergent_state_first_projection_at(
                List.cons(head, tail), pn, cn, pd, cd)
        }
        continued_fraction_convergent_state_first_projection_for_current_denominators(
            List.cons(head, tail), pn, cn, pd)
    }
}

/// The denominator first-projection package advances over one coefficient.
theorem continued_fraction_convergent_state_first_projection_for_denominators_cons(
    head: Nat, tail: List[Nat], pn: Nat, cn: Nat
) {
    continued_fraction_convergent_state_first_projection_for_states(tail)
        implies
        continued_fraction_convergent_state_first_projection_for_denominators(
            List.cons(head, tail), pn, cn)
} by {
    if continued_fraction_convergent_state_first_projection_for_states(tail) {
        forall(pd: Nat) {
            continued_fraction_convergent_state_first_projection_for_current_denominators_cons(
                head, tail, pn, cn, pd)
        }
        continued_fraction_convergent_state_first_projection_for_denominators(
            List.cons(head, tail), pn, cn)
    }
}

/// The current-numerator first-projection package advances over one
/// coefficient.
theorem continued_fraction_convergent_state_first_projection_for_current_numerators_cons(
    head: Nat, tail: List[Nat], pn: Nat
) {
    continued_fraction_convergent_state_first_projection_for_states(tail)
        implies
        continued_fraction_convergent_state_first_projection_for_current_numerators(
            List.cons(head, tail), pn)
} by {
    if continued_fraction_convergent_state_first_projection_for_states(tail) {
        forall(cn: Nat) {
            continued_fraction_convergent_state_first_projection_for_denominators_cons(
                head, tail, pn, cn)
        }
        continued_fraction_convergent_state_first_projection_for_current_numerators(
            List.cons(head, tail), pn)
    }
}

/// The state first-projection package advances over one coefficient.
theorem continued_fraction_convergent_state_first_projection_for_states_cons(
    head: Nat, tail: List[Nat]
) {
    continued_fraction_convergent_state_first_projection_for_states(tail)
        implies
        continued_fraction_convergent_state_first_projection_for_states(
            List.cons(head, tail))
} by {
    if continued_fraction_convergent_state_first_projection_for_states(tail) {
        forall(pn: Nat) {
            continued_fraction_convergent_state_first_projection_for_current_numerators_cons(
                head, tail, pn)
        }
        continued_fraction_convergent_state_first_projection_for_states(
            List.cons(head, tail))
    }
}

/// Every convergent state satisfies the first-projection continuant-state
/// property.
theorem continued_fraction_convergent_state_first_projection_for_states_all {
    forall(coefficients: List[Nat]) {
        continued_fraction_convergent_state_first_projection_for_states(coefficients)
    }
} by {
    continued_fraction_convergent_state_first_projection_for_states_nil
    forall(head: Nat, tail: List[Nat]) {
        if continued_fraction_convergent_state_first_projection_for_states(tail) {
            continued_fraction_convergent_state_first_projection_for_states_cons(
                head, tail)
            continued_fraction_convergent_state_first_projection_for_states(
                List.cons(head, tail))
        }
        continued_fraction_convergent_state_first_projection_for_states(tail) implies continued_fraction_convergent_state_first_projection_for_states(List.cons(head, tail))
    }
    List.induction(function(coefficients: List[Nat]) {
        continued_fraction_convergent_state_first_projection_for_states(coefficients)
    })
}

/// Every convergent state satisfies the first-projection continuant-state
/// predicate.
theorem continued_fraction_convergent_state_first_projection_at_all(
    coefficients: List[Nat], pn: Nat, cn: Nat, pd: Nat, cd: Nat
) {
    continued_fraction_convergent_state_first_projection_at(
        coefficients, pn, cn, pd, cd)
} by {
    continued_fraction_convergent_state_first_projection_for_states_all
    continued_fraction_convergent_state_first_projection_for_states_at(
        coefficients, pn, cn, pd, cd)
}

/// True if all current denominator states satisfy the second-projection
/// continuant-state property.
define continued_fraction_convergent_state_second_projection_for_current_denominators(
    coefficients: List[Nat], pn: Nat, cn: Nat, pd: Nat
) -> Bool {
    forall(cd: Nat) {
        continued_fraction_convergent_state_second_projection_at(
            coefficients, pn, cn, pd, cd)
    }
}

/// True if all denominator states satisfy the second-projection
/// continuant-state property.
define continued_fraction_convergent_state_second_projection_for_denominators(
    coefficients: List[Nat], pn: Nat, cn: Nat
) -> Bool {
    forall(pd: Nat) {
        continued_fraction_convergent_state_second_projection_for_current_denominators(
            coefficients, pn, cn, pd)
    }
}

/// True if all current numerator states satisfy the second-projection
/// continuant-state property.
define continued_fraction_convergent_state_second_projection_for_current_numerators(
    coefficients: List[Nat], pn: Nat
) -> Bool {
    forall(cn: Nat) {
        continued_fraction_convergent_state_second_projection_for_denominators(
            coefficients, pn, cn)
    }
}

/// True if every numerator-denominator state satisfies the second-projection
/// continuant-state property.
define continued_fraction_convergent_state_second_projection_for_states(
    coefficients: List[Nat]
) -> Bool {
    forall(pn: Nat) {
        continued_fraction_convergent_state_second_projection_for_current_numerators(
            coefficients, pn)
    }
}

/// A current-denominator package gives the corresponding second-projection
/// predicate.
theorem continued_fraction_convergent_state_second_projection_for_current_denominators_at(
    coefficients: List[Nat], pn: Nat, cn: Nat, pd: Nat, cd: Nat
) {
    continued_fraction_convergent_state_second_projection_for_current_denominators(
        coefficients, pn, cn, pd)
        implies continued_fraction_convergent_state_second_projection_at(
            coefficients, pn, cn, pd, cd)
} by {
    if continued_fraction_convergent_state_second_projection_for_current_denominators(
        coefficients, pn, cn, pd) {
        continued_fraction_convergent_state_second_projection_for_current_denominators(
            coefficients, pn, cn, pd) =
            forall(current_denominator: Nat) {
                continued_fraction_convergent_state_second_projection_at(
                    coefficients, pn, cn, pd, current_denominator)
            }
        let h: Bool = continued_fraction_convergent_state_second_projection_at(
            coefficients, pn, cn, pd, cd)
        continued_fraction_convergent_state_second_projection_at(
            coefficients, pn, cn, pd, cd)
    }
}

/// A denominator package gives the corresponding current-denominator package.
theorem continued_fraction_convergent_state_second_projection_for_denominators_at(
    coefficients: List[Nat], pn: Nat, cn: Nat, pd: Nat
) {
    continued_fraction_convergent_state_second_projection_for_denominators(
        coefficients, pn, cn)
        implies
        continued_fraction_convergent_state_second_projection_for_current_denominators(
            coefficients, pn, cn, pd)
} by {
    if continued_fraction_convergent_state_second_projection_for_denominators(
        coefficients, pn, cn) {
        continued_fraction_convergent_state_second_projection_for_denominators(
            coefficients, pn, cn) =
            forall(previous_denominator: Nat) {
                continued_fraction_convergent_state_second_projection_for_current_denominators(
                    coefficients, pn, cn, previous_denominator)
            }
        let h: Bool =
            continued_fraction_convergent_state_second_projection_for_current_denominators(
                coefficients, pn, cn, pd)
        continued_fraction_convergent_state_second_projection_for_current_denominators(
            coefficients, pn, cn, pd)
    }
}

/// A current-numerator package gives the corresponding denominator package.
theorem continued_fraction_convergent_state_second_projection_for_current_numerators_at(
    coefficients: List[Nat], pn: Nat, cn: Nat
) {
    continued_fraction_convergent_state_second_projection_for_current_numerators(
        coefficients, pn)
        implies continued_fraction_convergent_state_second_projection_for_denominators(
            coefficients, pn, cn)
} by {
    if continued_fraction_convergent_state_second_projection_for_current_numerators(
        coefficients, pn) {
        continued_fraction_convergent_state_second_projection_for_current_numerators(
            coefficients, pn) =
            forall(current_numerator: Nat) {
                continued_fraction_convergent_state_second_projection_for_denominators(
                    coefficients, pn, current_numerator)
            }
        let h: Bool =
            continued_fraction_convergent_state_second_projection_for_denominators(
                coefficients, pn, cn)
        continued_fraction_convergent_state_second_projection_for_denominators(
            coefficients, pn, cn)
    }
}

/// A state package gives the corresponding current-numerator package.
theorem continued_fraction_convergent_state_second_projection_for_states_at_numerator(
    coefficients: List[Nat], pn: Nat
) {
    continued_fraction_convergent_state_second_projection_for_states(coefficients)
        implies
        continued_fraction_convergent_state_second_projection_for_current_numerators(
            coefficients, pn)
} by {
    if continued_fraction_convergent_state_second_projection_for_states(coefficients) {
        continued_fraction_convergent_state_second_projection_for_states(coefficients) =
            forall(previous_numerator: Nat) {
                continued_fraction_convergent_state_second_projection_for_current_numerators(
                    coefficients, previous_numerator)
            }
        let h: Bool =
            continued_fraction_convergent_state_second_projection_for_current_numerators(
                coefficients, pn)
        continued_fraction_convergent_state_second_projection_for_current_numerators(
            coefficients, pn)
    }
}

/// A state package gives the corresponding second-projection predicate.
theorem continued_fraction_convergent_state_second_projection_for_states_at(
    coefficients: List[Nat], pn: Nat, cn: Nat, pd: Nat, cd: Nat
) {
    continued_fraction_convergent_state_second_projection_for_states(coefficients)
        implies continued_fraction_convergent_state_second_projection_at(
            coefficients, pn, cn, pd, cd)
} by {
    if continued_fraction_convergent_state_second_projection_for_states(coefficients) {
        continued_fraction_convergent_state_second_projection_for_states_at_numerator(
            coefficients, pn)
        continued_fraction_convergent_state_second_projection_for_current_numerators_at(
            coefficients, pn, cn)
        continued_fraction_convergent_state_second_projection_for_denominators_at(
            coefficients, pn, cn, pd)
        continued_fraction_convergent_state_second_projection_for_current_denominators_at(
            coefficients, pn, cn, pd, cd)
        continued_fraction_convergent_state_second_projection_at(
            coefficients, pn, cn, pd, cd)
    }
}

/// The empty coefficient list satisfies the current-denominator
/// second-projection package.
theorem continued_fraction_convergent_state_second_projection_for_current_denominators_nil(
    pn: Nat, cn: Nat, pd: Nat
) {
    continued_fraction_convergent_state_second_projection_for_current_denominators(
        List.nil[Nat], pn, cn, pd)
} by {
    forall(cd: Nat) {
        continued_fraction_convergent_state_second_projection_at_nil(pn, cn, pd, cd)
    }
}

/// The empty coefficient list satisfies the denominator second-projection
/// package.
theorem continued_fraction_convergent_state_second_projection_for_denominators_nil(
    pn: Nat, cn: Nat
) {
    continued_fraction_convergent_state_second_projection_for_denominators(
        List.nil[Nat], pn, cn)
} by {
    forall(pd: Nat) {
        continued_fraction_convergent_state_second_projection_for_current_denominators_nil(
            pn, cn, pd)
    }
}

/// The empty coefficient list satisfies the current-numerator
/// second-projection package.
theorem continued_fraction_convergent_state_second_projection_for_current_numerators_nil(
    pn: Nat
) {
    continued_fraction_convergent_state_second_projection_for_current_numerators(
        List.nil[Nat], pn)
} by {
    forall(cn: Nat) {
        continued_fraction_convergent_state_second_projection_for_denominators_nil(
            pn, cn)
    }
}

/// The empty coefficient list satisfies the state second-projection package.
theorem continued_fraction_convergent_state_second_projection_for_states_nil {
    continued_fraction_convergent_state_second_projection_for_states(List.nil[Nat])
} by {
    forall(pn: Nat) {
        continued_fraction_convergent_state_second_projection_for_current_numerators_nil(pn)
    }
}

/// The current-denominator second-projection package advances over one
/// coefficient.
theorem continued_fraction_convergent_state_second_projection_for_current_denominators_cons(
    head: Nat, tail: List[Nat], pn: Nat, cn: Nat, pd: Nat
) {
    continued_fraction_convergent_state_second_projection_for_states(tail)
        implies
        continued_fraction_convergent_state_second_projection_for_current_denominators(
            List.cons(head, tail), pn, cn, pd)
} by {
    if continued_fraction_convergent_state_second_projection_for_states(tail) {
        forall(cd: Nat) {
            continued_fraction_convergent_state_second_projection_for_states_at(
                tail, cn, cn * head + pn, cd, cd * head + pd)
            continued_fraction_convergent_state_second_projection_at_cons(
                head, tail, pn, cn, pd, cd)
            continued_fraction_convergent_state_second_projection_at(
                List.cons(head, tail), pn, cn, pd, cd)
        }
        continued_fraction_convergent_state_second_projection_for_current_denominators(
            List.cons(head, tail), pn, cn, pd)
    }
}

/// The denominator second-projection package advances over one coefficient.
theorem continued_fraction_convergent_state_second_projection_for_denominators_cons(
    head: Nat, tail: List[Nat], pn: Nat, cn: Nat
) {
    continued_fraction_convergent_state_second_projection_for_states(tail)
        implies
        continued_fraction_convergent_state_second_projection_for_denominators(
            List.cons(head, tail), pn, cn)
} by {
    if continued_fraction_convergent_state_second_projection_for_states(tail) {
        forall(pd: Nat) {
            continued_fraction_convergent_state_second_projection_for_current_denominators_cons(
                head, tail, pn, cn, pd)
        }
        continued_fraction_convergent_state_second_projection_for_denominators(
            List.cons(head, tail), pn, cn)
    }
}

/// The current-numerator second-projection package advances over one
/// coefficient.
theorem continued_fraction_convergent_state_second_projection_for_current_numerators_cons(
    head: Nat, tail: List[Nat], pn: Nat
) {
    continued_fraction_convergent_state_second_projection_for_states(tail)
        implies
        continued_fraction_convergent_state_second_projection_for_current_numerators(
            List.cons(head, tail), pn)
} by {
    if continued_fraction_convergent_state_second_projection_for_states(tail) {
        forall(cn: Nat) {
            continued_fraction_convergent_state_second_projection_for_denominators_cons(
                head, tail, pn, cn)
        }
        continued_fraction_convergent_state_second_projection_for_current_numerators(
            List.cons(head, tail), pn)
    }
}

/// The state second-projection package advances over one coefficient.
theorem continued_fraction_convergent_state_second_projection_for_states_cons(
    head: Nat, tail: List[Nat]
) {
    continued_fraction_convergent_state_second_projection_for_states(tail)
        implies
        continued_fraction_convergent_state_second_projection_for_states(
            List.cons(head, tail))
} by {
    if continued_fraction_convergent_state_second_projection_for_states(tail) {
        forall(pn: Nat) {
            continued_fraction_convergent_state_second_projection_for_current_numerators_cons(
                head, tail, pn)
        }
        continued_fraction_convergent_state_second_projection_for_states(
            List.cons(head, tail))
    }
}

/// Every convergent state satisfies the second-projection continuant-state
/// property.
theorem continued_fraction_convergent_state_second_projection_for_states_all {
    forall(coefficients: List[Nat]) {
        continued_fraction_convergent_state_second_projection_for_states(coefficients)
    }
} by {
    continued_fraction_convergent_state_second_projection_for_states_nil
    forall(head: Nat, tail: List[Nat]) {
        if continued_fraction_convergent_state_second_projection_for_states(tail) {
            continued_fraction_convergent_state_second_projection_for_states_cons(
                head, tail)
            continued_fraction_convergent_state_second_projection_for_states(
                List.cons(head, tail))
        }
        continued_fraction_convergent_state_second_projection_for_states(tail) implies continued_fraction_convergent_state_second_projection_for_states(List.cons(head, tail))
    }
    List.induction(function(coefficients: List[Nat]) {
        continued_fraction_convergent_state_second_projection_for_states(coefficients)
    })
}

/// Every convergent state satisfies the second-projection continuant-state
/// predicate.
theorem continued_fraction_convergent_state_second_projection_at_all(
    coefficients: List[Nat], pn: Nat, cn: Nat, pd: Nat, cd: Nat
) {
    continued_fraction_convergent_state_second_projection_at(
        coefficients, pn, cn, pd, cd)
} by {
    continued_fraction_convergent_state_second_projection_for_states_all
    continued_fraction_convergent_state_second_projection_for_states_at(
        coefficients, pn, cn, pd, cd)
}

/// A singleton convergent state has its first projection equal to the numerator
/// continuant state.
theorem continued_fraction_convergent_state_first_projection_at_singleton(
    head: Nat, pn: Nat, cn: Nat, pd: Nat, cd: Nat
) {
    continued_fraction_convergent_state_first_projection_at(
        List.cons(head, List.nil[Nat]), pn, cn, pd, cd)
} by {
    continued_fraction_convergent_state_first_projection_at_nil(
        cn, cn * head + pn, cd, cd * head + pd)
    continued_fraction_convergent_state_first_projection_at_cons(head, List.nil[Nat],
        pn, cn, pd, cd)
}

/// A singleton convergent state has its second projection equal to the
/// denominator continuant state.
theorem continued_fraction_convergent_state_second_projection_at_singleton(
    head: Nat, pn: Nat, cn: Nat, pd: Nat, cd: Nat
) {
    continued_fraction_convergent_state_second_projection_at(
        List.cons(head, List.nil[Nat]), pn, cn, pd, cd)
} by {
    continued_fraction_convergent_state_second_projection_at_nil(
        cn, cn * head + pn, cd, cd * head + pd)
    continued_fraction_convergent_state_second_projection_at_cons(head, List.nil[Nat],
        pn, cn, pd, cd)
}

/// A two-coefficient convergent state has its first projection equal to the
/// numerator continuant state.
theorem continued_fraction_convergent_state_first_projection_at_pair(
    head: Nat, next: Nat, pn: Nat, cn: Nat, pd: Nat, cd: Nat
) {
    continued_fraction_convergent_state_first_projection_at(
        List.cons(head, List.cons(next, List.nil[Nat])), pn, cn, pd, cd)
} by {
    continued_fraction_convergent_state_first_projection_at_singleton(next,
        cn, cn * head + pn, cd, cd * head + pd)
    continued_fraction_convergent_state_first_projection_at_cons(head,
        List.cons(next, List.nil[Nat]), pn, cn, pd, cd)
}

/// A two-coefficient convergent state has its second projection equal to the
/// denominator continuant state.
theorem continued_fraction_convergent_state_second_projection_at_pair(
    head: Nat, next: Nat, pn: Nat, cn: Nat, pd: Nat, cd: Nat
) {
    continued_fraction_convergent_state_second_projection_at(
        List.cons(head, List.cons(next, List.nil[Nat])), pn, cn, pd, cd)
} by {
    continued_fraction_convergent_state_second_projection_at_singleton(next,
        cn, cn * head + pn, cd, cd * head + pd)
    continued_fraction_convergent_state_second_projection_at_cons(head,
        List.cons(next, List.nil[Nat]), pn, cn, pd, cd)
}

/// A three-coefficient convergent state has its first projection equal to the
/// numerator continuant state.
theorem continued_fraction_convergent_state_first_projection_at_triple(
    head: Nat, next: Nat, third: Nat, pn: Nat, cn: Nat, pd: Nat, cd: Nat
) {
    continued_fraction_convergent_state_first_projection_at(
        List.cons(head, List.cons(next, List.cons(third, List.nil[Nat]))),
        pn, cn, pd, cd)
} by {
    continued_fraction_convergent_state_first_projection_at_pair(next, third,
        cn, cn * head + pn, cd, cd * head + pd)
    continued_fraction_convergent_state_first_projection_at_cons(head,
        List.cons(next, List.cons(third, List.nil[Nat])), pn, cn, pd, cd)
}

/// A three-coefficient convergent state has its second projection equal to the
/// denominator continuant state.
theorem continued_fraction_convergent_state_second_projection_at_triple(
    head: Nat, next: Nat, third: Nat, pn: Nat, cn: Nat, pd: Nat, cd: Nat
) {
    continued_fraction_convergent_state_second_projection_at(
        List.cons(head, List.cons(next, List.cons(third, List.nil[Nat]))),
        pn, cn, pd, cd)
} by {
    continued_fraction_convergent_state_second_projection_at_pair(next, third,
        cn, cn * head + pn, cd, cd * head + pd)
    continued_fraction_convergent_state_second_projection_at_cons(head,
        List.cons(next, List.cons(third, List.nil[Nat])), pn, cn, pd, cd)
}

/// The empty continuant state is positive when the current state is positive.
theorem continuant_state_nil_positive(previous: Nat, current: Nat) {
    Nat.0 < current implies Nat.0 < continuant_state(List.nil[Nat], previous, current)
} by {
    if Nat.0 < current {
        continuant_state_nil(previous, current)
        Nat.0 < continuant_state(List.nil[Nat], previous, current)
    }
}

/// Positivity of the tail state gives positivity after consing a coefficient.
theorem continuant_state_cons_positive_of_tail_positive(head: Nat, tail: List[Nat],
    previous: Nat, current: Nat) {
    Nat.0 < continuant_state(tail, current, current * head + previous)
        implies Nat.0 < continuant_state(List.cons(head, tail), previous, current)
} by {
    if Nat.0 < continuant_state(tail, current, current * head + previous) {
        continuant_state_cons(head, tail, previous, current)
        Nat.0 < continuant_state(List.cons(head, tail), previous, current)
    }
}

/// A singleton positive tail has a positive continuant state.
theorem continuant_state_singleton_positive(head: Nat, previous: Nat, current: Nat) {
    Nat.0 < current and Nat.0 < head
        implies Nat.0 < continuant_state(List.cons(head, List.nil[Nat]), previous, current)
} by {
    if Nat.0 < current and Nat.0 < head {
        continuant_state_next_positive(previous, current, head)
        continuant_state_nil_positive(current, current * head + previous)
        continuant_state_cons_positive_of_tail_positive(head, List.nil[Nat],
            previous, current)
        Nat.0 < continuant_state(List.cons(head, List.nil[Nat]), previous, current)
    }
}

/// A two-coefficient positive tail has a positive continuant state.
theorem continuant_state_pair_positive(head: Nat, next: Nat,
    previous: Nat, current: Nat) {
    Nat.0 < current and Nat.0 < head and Nat.0 < next
        implies Nat.0 < continuant_state(
            List.cons(head, List.cons(next, List.nil[Nat])), previous, current)
} by {
    if Nat.0 < current and Nat.0 < head and Nat.0 < next {
        continuant_state_next_positive(previous, current, head)
        continuant_state_singleton_positive(next, current, current * head + previous)
        continuant_state_cons_positive_of_tail_positive(head, List.cons(next, List.nil[Nat]),
            previous, current)
        Nat.0 < continuant_state(List.cons(head, List.cons(next, List.nil[Nat])),
            previous, current)
    }
}

/// True if one positive continuant state stays positive along a tail.
define continuant_state_positive_at(coefficients: List[Nat],
    previous: Nat, current: Nat) -> Bool {
    Nat.0 < current and positive_continued_fraction_tail(coefficients)
        implies Nat.0 < continuant_state(coefficients, previous, current)
}

/// True if every positive continuant state stays positive along a tail.
define continuant_state_positive_for_tail(coefficients: List[Nat]) -> Bool {
    forall(previous: Nat, current: Nat) {
        continuant_state_positive_at(coefficients, previous, current)
    }
}

/// The empty tail preserves positivity of one continuant state.
theorem continuant_state_positive_at_nil(previous: Nat, current: Nat) {
    continuant_state_positive_at(List.nil[Nat], previous, current)
} by {
    if not continuant_state_positive_at(List.nil[Nat], previous, current) {
        continuant_state_positive_at(List.nil[Nat], previous, current) =
            (Nat.0 < current and positive_continued_fraction_tail(List.nil[Nat])
                implies Nat.0 < continuant_state(List.nil[Nat], previous, current))
        continuant_state_nil_positive(previous, current)
        false
    }
}

/// The empty tail preserves positivity of every continuant state.
theorem continuant_state_positive_for_tail_nil {
    continuant_state_positive_for_tail(List.nil[Nat])
} by {
    forall(previous: Nat, current: Nat) {
        continuant_state_positive_at_nil(previous, current)
    }
}

/// A packaged positive-tail theorem gives the corresponding state predicate.
theorem continuant_state_positive_for_tail_at(coefficients: List[Nat],
    previous: Nat, current: Nat) {
    continuant_state_positive_for_tail(coefficients)
        implies continuant_state_positive_at(coefficients, previous, current)
} by {
    if continuant_state_positive_for_tail(coefficients) {
        continuant_state_positive_for_tail(coefficients) =
            forall(prev: Nat, curr: Nat) {
                continuant_state_positive_at(coefficients, prev, curr)
        }
        let h: Bool = continuant_state_positive_at(coefficients, previous, current)
        continuant_state_positive_at(coefficients, previous, current)
    }
}

/// A state predicate gives the corresponding state inequality.
theorem continuant_state_positive_at_elim(coefficients: List[Nat],
    previous: Nat, current: Nat) {
    continuant_state_positive_at(coefficients, previous, current)
        and Nat.0 < current
        and positive_continued_fraction_tail(coefficients)
        implies Nat.0 < continuant_state(coefficients, previous, current)
} by {
    if continuant_state_positive_at(coefficients, previous, current)
        and Nat.0 < current
        and positive_continued_fraction_tail(coefficients) {
        if not Nat.0 < continuant_state(coefficients, previous, current) {
            continuant_state_positive_at(coefficients, previous, current) =
                (Nat.0 < current and positive_continued_fraction_tail(coefficients)
                    implies Nat.0 < continuant_state(coefficients, previous, current))
            false
        }
    }
}

/// Positive-state preservation advances over one positive coefficient.
theorem continuant_state_positive_at_cons(head: Nat, tail: List[Nat],
    previous: Nat, current: Nat) {
    continuant_state_positive_for_tail(tail)
        implies continuant_state_positive_at(List.cons(head, tail), previous, current)
} by {
    if continuant_state_positive_for_tail(tail) {
        if Nat.0 < current
            and positive_continued_fraction_tail(List.cons(head, tail)) {
            positive_continued_fraction_tail_cons_head(head, tail)
            positive_continued_fraction_tail_cons_tail(head, tail)
            continuant_state_next_positive(previous, current, head)
            Nat.0 < current * head + previous
            continuant_state_positive_for_tail_at(tail, current,
                current * head + previous)
            continuant_state_positive_at_elim(tail, current,
                current * head + previous)
            continuant_state_cons_positive_of_tail_positive(head, tail,
                previous, current)
            Nat.0 < continuant_state(List.cons(head, tail), previous, current)
        }
        continuant_state_positive_at(List.cons(head, tail), previous, current)
    }
}

/// Positive-state preservation advances over one positive coefficient.
theorem continuant_state_positive_for_tail_cons(head: Nat, tail: List[Nat]) {
    continuant_state_positive_for_tail(tail)
        implies continuant_state_positive_for_tail(List.cons(head, tail))
} by {
    if continuant_state_positive_for_tail(tail) {
        forall(previous: Nat, current: Nat) {
            continuant_state_positive_at_cons(head, tail, previous, current)
        }
        continuant_state_positive_for_tail(List.cons(head, tail))
    }
}

/// Every positive tail preserves positivity of every continuant state.
theorem continuant_state_positive_for_tail_all {
    forall(coefficients: List[Nat]) {
        continuant_state_positive_for_tail(coefficients)
    }
} by {
    continuant_state_positive_for_tail_nil
    forall(head: Nat, tail: List[Nat]) {
        if continuant_state_positive_for_tail(tail) {
            continuant_state_positive_for_tail_cons(head, tail)
            continuant_state_positive_for_tail(List.cons(head, tail))
        }
        continuant_state_positive_for_tail(tail) implies continuant_state_positive_for_tail(List.cons(head, tail))
    }
    List.induction(function(l: List[Nat]) { continuant_state_positive_for_tail(l) })
}

/// A positive tail preserves positivity of every continuant state.
theorem continuant_state_positive_of_positive_tail(coefficients: List[Nat],
    previous: Nat, current: Nat) {
    Nat.0 < current and positive_continued_fraction_tail(coefficients)
        implies Nat.0 < continuant_state(coefficients, previous, current)
} by {
    if Nat.0 < current and positive_continued_fraction_tail(coefficients) {
        continuant_state_positive_for_tail_all
        continuant_state_positive_for_tail(coefficients)
        continuant_state_positive_for_tail_at(coefficients, previous, current)
        continuant_state_positive_at_elim(coefficients, previous, current)
    }
}

/// A positive tail has positive continuant.
theorem continuant_positive_of_positive_tail(coefficients: List[Nat]) {
    positive_continued_fraction_tail(coefficients)
        implies Nat.0 < continuant(coefficients)
} by {
    if positive_continued_fraction_tail(coefficients) {
        continuant_state_positive_of_positive_tail(coefficients, Nat.0, Nat.1)
        Nat.0 < continuant_state(coefficients, Nat.0, Nat.1)
        Nat.0 < continuant(coefficients)
    }
}

/// A positive tail has nonzero continuant.
theorem continuant_ne_zero_of_positive_tail(coefficients: List[Nat]) {
    positive_continued_fraction_tail(coefficients)
        implies continuant(coefficients) != Nat.0
} by {
    if positive_continued_fraction_tail(coefficients) {
        continuant_positive_of_positive_tail(coefficients)
        nat_positive_ne_zero(continuant(coefficients))
    }
}

/// The denominator of a nonempty coefficient list is the continuant of its
/// tail.
theorem continued_fraction_denominator_cons(head: Nat, tail: List[Nat]) {
    continued_fraction_denominator(List.cons(head, tail)) = continuant(tail)
}

/// A nonempty coefficient list with positive tail has positive denominator.
theorem continued_fraction_denominator_cons_positive_of_tail_positive(
    head: Nat, tail: List[Nat]
) {
    positive_continued_fraction_tail(tail)
        implies Nat.0 < continued_fraction_denominator(List.cons(head, tail))
} by {
    if positive_continued_fraction_tail(tail) {
        continuant_positive_of_positive_tail(tail)
        continued_fraction_denominator_cons(head, tail)
        Nat.0 < continued_fraction_denominator(List.cons(head, tail))
    }
}

/// A valid nonempty coefficient list has positive denominator.
theorem continued_fraction_denominator_cons_positive(head: Nat, tail: List[Nat]) {
    finite_continued_fraction_coefficients(List.cons(head, tail))
        implies Nat.0 < continued_fraction_denominator(List.cons(head, tail))
} by {
    if finite_continued_fraction_coefficients(List.cons(head, tail)) {
        finite_continued_fraction_coefficients_cons_tail(head, tail)
        positive_continued_fraction_tail(tail)
        continued_fraction_denominator_cons_positive_of_tail_positive(head, tail)
    }
}

/// A nonempty coefficient list with positive tail has nonzero denominator.
theorem continued_fraction_denominator_cons_ne_zero_of_tail_positive(
    head: Nat, tail: List[Nat]
) {
    positive_continued_fraction_tail(tail)
        implies continued_fraction_denominator(List.cons(head, tail)) != Nat.0
} by {
    if positive_continued_fraction_tail(tail) {
        continued_fraction_denominator_cons_positive_of_tail_positive(head, tail)
        nat_positive_ne_zero(continued_fraction_denominator(List.cons(head, tail)))
    }
}

/// A valid nonempty coefficient list has nonzero denominator.
theorem continued_fraction_denominator_cons_ne_zero(head: Nat, tail: List[Nat]) {
    finite_continued_fraction_coefficients(List.cons(head, tail))
        implies continued_fraction_denominator(List.cons(head, tail)) != Nat.0
} by {
    if finite_continued_fraction_coefficients(List.cons(head, tail)) {
        continued_fraction_denominator_cons_positive(head, tail)
        nat_positive_ne_zero(continued_fraction_denominator(List.cons(head, tail)))
    }
}

/// Every valid coefficient list has positive denominator.
theorem continued_fraction_denominator_positive_of_valid_coefficients(
    coefficients: List[Nat]
) {
    finite_continued_fraction_coefficients(coefficients)
        implies Nat.0 < continued_fraction_denominator(coefficients)
} by {
    if finite_continued_fraction_coefficients(coefficients) {
        match coefficients {
            List.nil {
            }
            List.cons(head, tail) {
                continued_fraction_denominator_cons_positive(head, tail)
                Nat.0 < continued_fraction_denominator(List.cons(head, tail))
                Nat.0 < continued_fraction_denominator(coefficients)
            }
        }
    }
}

/// Every valid coefficient list has nonzero denominator.
theorem continued_fraction_denominator_ne_zero_of_valid_coefficients(
    coefficients: List[Nat]
) {
    finite_continued_fraction_coefficients(coefficients)
        implies continued_fraction_denominator(coefficients) != Nat.0
} by {
    if finite_continued_fraction_coefficients(coefficients) {
        continued_fraction_denominator_positive_of_valid_coefficients(coefficients)
        nat_positive_ne_zero(continued_fraction_denominator(coefficients))
    }
}

/// The denominator of a continued fraction is positive.
theorem continued_fraction_denominator_positive(cf: ContinuedFraction) {
    Nat.0 < cf.denominator
} by {
    continued_fraction_coefficients_valid(cf)
    continued_fraction_denominator_positive_of_valid_coefficients(cf.coefficients)
    continued_fraction_denominator_eq_coefficients_denominator(cf)
}

/// The denominator of a continued fraction is nonzero.
theorem continued_fraction_denominator_ne_zero(cf: ContinuedFraction) {
    cf.denominator != Nat.0
} by {
    continued_fraction_denominator_positive(cf)
    nat_positive_ne_zero(cf.denominator)
}

/// A continuant of appended positive tails is positive.
theorem continuant_append_positive_of_positive_tails(left: List[Nat], right: List[Nat]) {
    positive_continued_fraction_tail(left) and positive_continued_fraction_tail(right)
        implies Nat.0 < continuant(left + right)
} by {
    if positive_continued_fraction_tail(left) and positive_continued_fraction_tail(right) {
        positive_continued_fraction_tail_append(left, right)
        positive_continued_fraction_tail(left + right)
        continuant_positive_of_positive_tail(left + right)
    }
}

/// A continuant of appended positive tails is nonzero.
theorem continuant_append_ne_zero_of_positive_tails(left: List[Nat], right: List[Nat]) {
    positive_continued_fraction_tail(left) and positive_continued_fraction_tail(right)
        implies continuant(left + right) != Nat.0
} by {
    if positive_continued_fraction_tail(left) and positive_continued_fraction_tail(right) {
        continuant_append_positive_of_positive_tails(left, right)
        nat_positive_ne_zero(continuant(left + right))
    }
}

/// A denominator whose tail is an append of positive tails is positive.
theorem continued_fraction_denominator_cons_append_positive_of_positive_tails(
    head: Nat, left_tail: List[Nat], right_tail: List[Nat]
) {
    positive_continued_fraction_tail(left_tail)
        and positive_continued_fraction_tail(right_tail)
        implies Nat.0 < continued_fraction_denominator(
            List.cons(head, left_tail + right_tail))
} by {
    if positive_continued_fraction_tail(left_tail)
        and positive_continued_fraction_tail(right_tail) {
        positive_continued_fraction_tail_append(left_tail, right_tail)
        positive_continued_fraction_tail(left_tail + right_tail)
        continued_fraction_denominator_cons_positive_of_tail_positive(
            head, left_tail + right_tail)
    }
}

/// A denominator whose tail is an append of positive tails is nonzero.
theorem continued_fraction_denominator_cons_append_ne_zero_of_positive_tails(
    head: Nat, left_tail: List[Nat], right_tail: List[Nat]
) {
    positive_continued_fraction_tail(left_tail)
        and positive_continued_fraction_tail(right_tail)
        implies continued_fraction_denominator(
            List.cons(head, left_tail + right_tail)) != Nat.0
} by {
    if positive_continued_fraction_tail(left_tail)
        and positive_continued_fraction_tail(right_tail) {
        continued_fraction_denominator_cons_append_positive_of_positive_tails(
            head, left_tail, right_tail)
        nat_positive_ne_zero(continued_fraction_denominator(
            List.cons(head, left_tail + right_tail)))
    }
}

/// A continued fraction whose tail is an append of positive tails has positive
/// denominator.
theorem continued_fraction_append_tail_denominator_positive(
    cf: ContinuedFraction, head: Nat, left_tail: List[Nat], right_tail: List[Nat]
) {
    cf.coefficients = List.cons(head, left_tail + right_tail)
        and positive_continued_fraction_tail(left_tail)
        and positive_continued_fraction_tail(right_tail)
        implies Nat.0 < cf.denominator
} by {
    if cf.coefficients = List.cons(head, left_tail + right_tail)
        and positive_continued_fraction_tail(left_tail)
        and positive_continued_fraction_tail(right_tail) {
        continued_fraction_denominator_cons_append_positive_of_positive_tails(
            head, left_tail, right_tail)
        continued_fraction_denominator_eq_coefficients_denominator(cf)
        Nat.0 < cf.denominator
    }
}

/// A continued fraction whose tail is an append of positive tails has nonzero
/// denominator.
theorem continued_fraction_append_tail_denominator_ne_zero(
    cf: ContinuedFraction, head: Nat, left_tail: List[Nat], right_tail: List[Nat]
) {
    cf.coefficients = List.cons(head, left_tail + right_tail)
        and positive_continued_fraction_tail(left_tail)
        and positive_continued_fraction_tail(right_tail)
        implies cf.denominator != Nat.0
} by {
    if cf.coefficients = List.cons(head, left_tail + right_tail)
        and positive_continued_fraction_tail(left_tail)
        and positive_continued_fraction_tail(right_tail) {
        continued_fraction_append_tail_denominator_positive(
            cf, head, left_tail, right_tail)
        nat_positive_ne_zero(cf.denominator)
    }
}

/// The first projection of a singleton convergent state is the next numerator
/// state.
theorem continued_fraction_convergent_state_singleton_first(head: Nat,
    pn: Nat, cn: Nat, pd: Nat, cd: Nat) {
    continued_fraction_convergent_state(List.cons(head, List.nil[Nat]),
        pn, cn, pd, cd).first = cn * head + pn
} by {
    continued_fraction_convergent_state_first_projection_at_singleton(head,
        pn, cn, pd, cd)
    continued_fraction_convergent_state_first_eq_of_projection_at(
        List.cons(head, List.nil[Nat]), pn, cn, pd, cd)
    continuant_state_singleton(head, pn, cn)
}

/// The second projection of a singleton convergent state is the next
/// denominator state.
theorem continued_fraction_convergent_state_singleton_second(head: Nat,
    pn: Nat, cn: Nat, pd: Nat, cd: Nat) {
    continued_fraction_convergent_state(List.cons(head, List.nil[Nat]),
        pn, cn, pd, cd).second = cd * head + pd
} by {
    continued_fraction_convergent_state_second_projection_at_singleton(head,
        pn, cn, pd, cd)
    continued_fraction_convergent_state_second_eq_of_projection_at(
        List.cons(head, List.nil[Nat]), pn, cn, pd, cd)
    continuant_state_singleton(head, pd, cd)
}

/// The first projection of a two-coefficient convergent state is the numerator
/// recurrence after two steps.
theorem continued_fraction_convergent_state_pair_first(head: Nat, next: Nat,
    pn: Nat, cn: Nat, pd: Nat, cd: Nat) {
    continued_fraction_convergent_state(List.cons(head, List.cons(next, List.nil[Nat])),
        pn, cn, pd, cd).first =
        (cn * head + pn) * next + cn
} by {
    continued_fraction_convergent_state_first_projection_at_pair(head, next,
        pn, cn, pd, cd)
    continued_fraction_convergent_state_first_eq_of_projection_at(
        List.cons(head, List.cons(next, List.nil[Nat])), pn, cn, pd, cd)
    continuant_state_pair(head, next, pn, cn)
}

/// The second projection of a two-coefficient convergent state is the
/// denominator recurrence after two steps.
theorem continued_fraction_convergent_state_pair_second(head: Nat, next: Nat,
    pn: Nat, cn: Nat, pd: Nat, cd: Nat) {
    continued_fraction_convergent_state(List.cons(head, List.cons(next, List.nil[Nat])),
        pn, cn, pd, cd).second =
        (cd * head + pd) * next + cd
} by {
    continued_fraction_convergent_state_second_projection_at_pair(head, next,
        pn, cn, pd, cd)
    continued_fraction_convergent_state_second_eq_of_projection_at(
        List.cons(head, List.cons(next, List.nil[Nat])), pn, cn, pd, cd)
    continuant_state_pair(head, next, pd, cd)
}

/// The first projection of a three-coefficient convergent state is the
/// numerator recurrence after three steps.
theorem continued_fraction_convergent_state_triple_first(head: Nat, next: Nat,
    third: Nat, pn: Nat, cn: Nat, pd: Nat, cd: Nat) {
    continued_fraction_convergent_state(
        List.cons(head, List.cons(next, List.cons(third, List.nil[Nat]))),
        pn, cn, pd, cd).first =
        ((cn * head + pn) * next + cn) * third + (cn * head + pn)
} by {
    continued_fraction_convergent_state_first_projection_at_triple(head, next, third,
        pn, cn, pd, cd)
    continued_fraction_convergent_state_first_eq_of_projection_at(
        List.cons(head, List.cons(next, List.cons(third, List.nil[Nat]))),
        pn, cn, pd, cd)
    continuant_state_triple(head, next, third, pn, cn)
}

/// The second projection of a three-coefficient convergent state is the
/// denominator recurrence after three steps.
theorem continued_fraction_convergent_state_triple_second(head: Nat, next: Nat,
    third: Nat, pn: Nat, cn: Nat, pd: Nat, cd: Nat) {
    continued_fraction_convergent_state(
        List.cons(head, List.cons(next, List.cons(third, List.nil[Nat]))),
        pn, cn, pd, cd).second =
        ((cd * head + pd) * next + cd) * third + (cd * head + pd)
} by {
    continued_fraction_convergent_state_second_projection_at_triple(head, next, third,
        pn, cn, pd, cd)
    continued_fraction_convergent_state_second_eq_of_projection_at(
        List.cons(head, List.cons(next, List.cons(third, List.nil[Nat]))),
        pn, cn, pd, cd)
    continuant_state_triple(head, next, third, pd, cd)
}

/// A first-projection predicate for a nonempty coefficient list identifies the
/// convergent numerator with the list numerator.
theorem continued_fraction_convergent_cons_first_eq_numerator_of_projection_at(
    head: Nat, tail: List[Nat]
) {
    continued_fraction_convergent_state_first_projection_at(
        List.cons(head, tail), Nat.0, Nat.1, Nat.1, Nat.0)
        implies continued_fraction_convergent(List.cons(head, tail)).first =
            continued_fraction_numerator(List.cons(head, tail))
} by {
    if continued_fraction_convergent_state_first_projection_at(
        List.cons(head, tail), Nat.0, Nat.1, Nat.1, Nat.0) {
        continued_fraction_convergent_state_first_eq_of_projection_at(
            List.cons(head, tail), Nat.0, Nat.1, Nat.1, Nat.0)
        continuant(List.cons(head, tail)) =
            continuant_state(List.cons(head, tail), Nat.0, Nat.1)
        continued_fraction_convergent(List.cons(head, tail)).first =
            continued_fraction_numerator(List.cons(head, tail))
    }
}

/// A second-projection predicate for a nonempty coefficient list identifies the
/// convergent denominator with the list denominator.
theorem continued_fraction_convergent_cons_second_eq_denominator_of_projection_at(
    head: Nat, tail: List[Nat]
) {
    continued_fraction_convergent_state_second_projection_at(
        List.cons(head, tail), Nat.0, Nat.1, Nat.1, Nat.0)
        implies continued_fraction_convergent(List.cons(head, tail)).second =
            continued_fraction_denominator(List.cons(head, tail))
} by {
    if continued_fraction_convergent_state_second_projection_at(
        List.cons(head, tail), Nat.0, Nat.1, Nat.1, Nat.0) {
        continued_fraction_convergent_state_second_eq_of_projection_at(
            List.cons(head, tail), Nat.0, Nat.1, Nat.1, Nat.0)
        continuant_state_cons(head, tail, Nat.1, Nat.0)
        mul_zero_right(head)
        continuant_state(tail, Nat.0, Nat.0 * head + Nat.1) =
            continuant_state(tail, Nat.0, Nat.1)
        continuant(tail) = continuant_state(tail, Nat.0, Nat.1)
        continued_fraction_convergent(List.cons(head, tail)).second =
            continued_fraction_denominator(List.cons(head, tail))
    }
}

/// A nonempty convergent has first projection equal to the numerator.
theorem continued_fraction_convergent_cons_first_eq_numerator(
    head: Nat, tail: List[Nat]
) {
    continued_fraction_convergent(List.cons(head, tail)).first =
        continued_fraction_numerator(List.cons(head, tail))
} by {
    continued_fraction_convergent_state_first_projection_at_all(
        List.cons(head, tail), Nat.0, Nat.1, Nat.1, Nat.0)
    continued_fraction_convergent_cons_first_eq_numerator_of_projection_at(head, tail)
}

/// A nonempty convergent has second projection equal to the denominator.
theorem continued_fraction_convergent_cons_second_eq_denominator(
    head: Nat, tail: List[Nat]
) {
    continued_fraction_convergent(List.cons(head, tail)).second =
        continued_fraction_denominator(List.cons(head, tail))
} by {
    continued_fraction_convergent_state_second_projection_at_all(
        List.cons(head, tail), Nat.0, Nat.1, Nat.1, Nat.0)
    continued_fraction_convergent_cons_second_eq_denominator_of_projection_at(head, tail)
}

/// Every convergent has first projection equal to the numerator.
theorem continued_fraction_convergent_first_eq_numerator(coefficients: List[Nat]) {
    continued_fraction_convergent(coefficients).first =
        continued_fraction_numerator(coefficients)
} by {
    match coefficients {
        List.nil {
            continued_fraction_convergent_nil_first_eq_numerator
            continued_fraction_convergent(coefficients).first =
                continued_fraction_numerator(coefficients)
        }
        List.cons(head, tail) {
            continued_fraction_convergent_cons_first_eq_numerator(head, tail)
            continued_fraction_convergent(List.cons(head, tail)).first =
                continued_fraction_numerator(List.cons(head, tail))
            continued_fraction_convergent(coefficients).first =
                continued_fraction_numerator(coefficients)
        }
    }
}

/// Every convergent has second projection equal to the denominator.
theorem continued_fraction_convergent_second_eq_denominator(coefficients: List[Nat]) {
    continued_fraction_convergent(coefficients).second =
        continued_fraction_denominator(coefficients)
} by {
    match coefficients {
        List.nil {
            continued_fraction_convergent_nil_second_eq_denominator
            continued_fraction_convergent(List.nil[Nat]).second =
                continued_fraction_denominator(List.nil[Nat])
            continued_fraction_convergent(coefficients).second =
                continued_fraction_denominator(coefficients)
        }
        List.cons(head, tail) {
            continued_fraction_convergent_cons_second_eq_denominator(head, tail)
            continued_fraction_convergent(List.cons(head, tail)).second =
                continued_fraction_denominator(List.cons(head, tail))
            continued_fraction_convergent(coefficients).second =
                continued_fraction_denominator(coefficients)
        }
    }
}

/// The numerator is the first projection of the convergent.
theorem continued_fraction_numerator_eq_convergent_first(coefficients: List[Nat]) {
    continued_fraction_numerator(coefficients) =
        continued_fraction_convergent(coefficients).first
} by {
    continued_fraction_convergent_first_eq_numerator(coefficients)
}

/// The denominator is the second projection of the convergent.
theorem continued_fraction_denominator_eq_convergent_second(coefficients: List[Nat]) {
    continued_fraction_denominator(coefficients) =
        continued_fraction_convergent(coefficients).second
} by {
    continued_fraction_convergent_second_eq_denominator(coefficients)
}

/// Every convergent is the pair formed by its numerator and denominator.
theorem continued_fraction_convergent_eq_numerator_denominator_pair(
    coefficients: List[Nat]
) {
    continued_fraction_convergent(coefficients) =
        Pair.new(continued_fraction_numerator(coefficients),
            continued_fraction_denominator(coefficients))
} by {
    continued_fraction_convergent_first_eq_numerator(coefficients)
    continued_fraction_convergent_second_eq_denominator(coefficients)
    pair_new_first(continued_fraction_numerator(coefficients),
        continued_fraction_denominator(coefficients))
    pair_new_second(continued_fraction_numerator(coefficients),
        continued_fraction_denominator(coefficients))
    pair_ext(continued_fraction_convergent(coefficients),
        Pair.new(continued_fraction_numerator(coefficients),
            continued_fraction_denominator(coefficients)))
}

/// The convergent method has first projection equal to the numerator method.
theorem continued_fraction_convergent_first_eq_numerator_method(cf: ContinuedFraction) {
    cf.convergent.first = cf.numerator
} by {
    continued_fraction_convergent_eq_coefficients_convergent(cf)
    continued_fraction_numerator_eq_coefficients_numerator(cf)
    continued_fraction_convergent_first_eq_numerator(cf.coefficients)
}

/// The convergent method has second projection equal to the denominator method.
theorem continued_fraction_convergent_second_eq_denominator_method(cf: ContinuedFraction) {
    cf.convergent.second = cf.denominator
} by {
    continued_fraction_convergent_eq_coefficients_convergent(cf)
    continued_fraction_denominator_eq_coefficients_denominator(cf)
    continued_fraction_convergent_second_eq_denominator(cf.coefficients)
}

/// The numerator method is the first projection of the convergent method.
theorem continued_fraction_numerator_method_eq_convergent_first(cf: ContinuedFraction) {
    cf.numerator = cf.convergent.first
} by {
    continued_fraction_convergent_first_eq_numerator_method(cf)
}

/// The denominator method is the second projection of the convergent method.
theorem continued_fraction_denominator_method_eq_convergent_second(cf: ContinuedFraction) {
    cf.denominator = cf.convergent.second
} by {
    continued_fraction_convergent_second_eq_denominator_method(cf)
}

/// The convergent method is the pair formed by the numerator and denominator
/// methods.
theorem continued_fraction_convergent_eq_numerator_denominator_method_pair(
    cf: ContinuedFraction
) {
    cf.convergent = Pair.new(cf.numerator, cf.denominator)
} by {
    continued_fraction_convergent_first_eq_numerator_method(cf)
    continued_fraction_convergent_second_eq_denominator_method(cf)
    pair_new_first(cf.numerator, cf.denominator)
    pair_new_second(cf.numerator, cf.denominator)
    pair_ext(cf.convergent, Pair.new(cf.numerator, cf.denominator))
}

/// A valid coefficient list has positive convergent denominator.
theorem continued_fraction_convergent_second_positive_of_valid_coefficients(
    coefficients: List[Nat]
) {
    finite_continued_fraction_coefficients(coefficients)
        implies Nat.0 < continued_fraction_convergent(coefficients).second
} by {
    if finite_continued_fraction_coefficients(coefficients) {
        continued_fraction_convergent_second_eq_denominator(coefficients)
        continued_fraction_denominator_positive_of_valid_coefficients(coefficients)
        Nat.0 < continued_fraction_denominator(coefficients)
        Nat.0 < continued_fraction_convergent(coefficients).second
    }
}

/// A valid coefficient list has nonzero convergent denominator.
theorem continued_fraction_convergent_second_ne_zero_of_valid_coefficients(
    coefficients: List[Nat]
) {
    finite_continued_fraction_coefficients(coefficients)
        implies continued_fraction_convergent(coefficients).second != Nat.0
} by {
    if finite_continued_fraction_coefficients(coefficients) {
        continued_fraction_convergent_second_positive_of_valid_coefficients(coefficients)
        nat_positive_ne_zero(continued_fraction_convergent(coefficients).second)
    }
}

/// A continued fraction has positive convergent denominator.
theorem continued_fraction_convergent_second_positive(cf: ContinuedFraction) {
    Nat.0 < cf.convergent.second
} by {
    continued_fraction_coefficients_valid(cf)
    continued_fraction_convergent_eq_coefficients_convergent(cf)
    continued_fraction_convergent_second_positive_of_valid_coefficients(cf.coefficients)
    Nat.0 < continued_fraction_convergent(cf.coefficients).second
    Nat.0 < cf.convergent.second
}

/// A continued fraction has nonzero convergent denominator.
theorem continued_fraction_convergent_second_ne_zero(cf: ContinuedFraction) {
    cf.convergent.second != Nat.0
} by {
    continued_fraction_convergent_second_positive(cf)
    nat_positive_ne_zero(cf.convergent.second)
}

/// The first projection of a convergent state is its numerator continuant
/// state.
theorem continued_fraction_convergent_state_first_eq_continuant_state(
    coefficients: List[Nat], pn: Nat, cn: Nat, pd: Nat, cd: Nat
) {
    continued_fraction_convergent_state(coefficients, pn, cn, pd, cd).first =
        continuant_state(coefficients, pn, cn)
} by {
    continued_fraction_convergent_state_first_projection_at_all(
        coefficients, pn, cn, pd, cd)
    continued_fraction_convergent_state_first_eq_of_projection_at(
        coefficients, pn, cn, pd, cd)
}

/// The second projection of a convergent state is its denominator continuant
/// state.
theorem continued_fraction_convergent_state_second_eq_continuant_state(
    coefficients: List[Nat], pn: Nat, cn: Nat, pd: Nat, cd: Nat
) {
    continued_fraction_convergent_state(coefficients, pn, cn, pd, cd).second =
        continuant_state(coefficients, pd, cd)
} by {
    continued_fraction_convergent_state_second_projection_at_all(
        coefficients, pn, cn, pd, cd)
    continued_fraction_convergent_state_second_eq_of_projection_at(
        coefficients, pn, cn, pd, cd)
}

/// Every convergent state is the pair of its two continuant states.
theorem continued_fraction_convergent_state_eq_continuant_state_pair(
    coefficients: List[Nat], pn: Nat, cn: Nat, pd: Nat, cd: Nat
) {
    continued_fraction_convergent_state(coefficients, pn, cn, pd, cd) =
        Pair.new(continuant_state(coefficients, pn, cn),
            continuant_state(coefficients, pd, cd))
} by {
    continued_fraction_convergent_state_first_eq_continuant_state(
        coefficients, pn, cn, pd, cd)
    continued_fraction_convergent_state_second_eq_continuant_state(
        coefficients, pn, cn, pd, cd)
    pair_new_first(continuant_state(coefficients, pn, cn),
        continuant_state(coefficients, pd, cd))
    pair_new_second(continuant_state(coefficients, pn, cn),
        continuant_state(coefficients, pd, cd))
    pair_ext(continued_fraction_convergent_state(coefficients, pn, cn, pd, cd),
        Pair.new(continuant_state(coefficients, pn, cn),
            continuant_state(coefficients, pd, cd)))
}

/// A nonempty coefficient list has numerator equal to its continuant.
theorem continued_fraction_numerator_cons_eq_continuant(head: Nat, tail: List[Nat]) {
    continued_fraction_numerator(List.cons(head, tail)) =
        continuant(List.cons(head, tail))
}

/// A nonempty coefficient list has denominator equal to the tail continuant.
theorem continued_fraction_denominator_cons_eq_tail_continuant(
    head: Nat, tail: List[Nat]
) {
    continued_fraction_denominator(List.cons(head, tail)) = continuant(tail)
} by {
    continued_fraction_denominator_cons(head, tail)
}

/// A nonempty convergent has first projection equal to its continuant.
theorem continued_fraction_convergent_cons_first_eq_continuant(
    head: Nat, tail: List[Nat]
) {
    continued_fraction_convergent(List.cons(head, tail)).first =
        continuant(List.cons(head, tail))
} by {
    continued_fraction_convergent_cons_first_eq_numerator(head, tail)
    continued_fraction_numerator_cons_eq_continuant(head, tail)
}

/// A nonempty convergent has second projection equal to the tail continuant.
theorem continued_fraction_convergent_cons_second_eq_tail_continuant(
    head: Nat, tail: List[Nat]
) {
    continued_fraction_convergent(List.cons(head, tail)).second = continuant(tail)
} by {
    continued_fraction_convergent_cons_second_eq_denominator(head, tail)
    continued_fraction_denominator_cons_eq_tail_continuant(head, tail)
}

/// A nonempty convergent is the pair of the list continuant and tail
/// continuant.
theorem continued_fraction_convergent_cons_eq_continuant_pair(
    head: Nat, tail: List[Nat]
) {
    continued_fraction_convergent(List.cons(head, tail)) =
        Pair.new(continuant(List.cons(head, tail)), continuant(tail))
} by {
    continued_fraction_convergent_cons_first_eq_continuant(head, tail)
    continued_fraction_convergent_cons_second_eq_tail_continuant(head, tail)
    pair_new_first(continuant(List.cons(head, tail)), continuant(tail))
    pair_new_second(continuant(List.cons(head, tail)), continuant(tail))
    pair_ext(continued_fraction_convergent(List.cons(head, tail)),
        Pair.new(continuant(List.cons(head, tail)), continuant(tail)))
}

/// A continued fraction with displayed nonempty coefficients has numerator
/// equal to the displayed continuant.
theorem continued_fraction_cons_numerator_eq_continuant(
    cf: ContinuedFraction, head: Nat, tail: List[Nat]
) {
    cf.coefficients = List.cons(head, tail)
        implies cf.numerator = continuant(List.cons(head, tail))
} by {
    if cf.coefficients = List.cons(head, tail) {
        continued_fraction_numerator_eq_coefficients_numerator(cf)
        continued_fraction_numerator_cons_eq_continuant(head, tail)
    }
}

/// A continued fraction with displayed nonempty coefficients has denominator
/// equal to the tail continuant.
theorem continued_fraction_cons_denominator_eq_tail_continuant(
    cf: ContinuedFraction, head: Nat, tail: List[Nat]
) {
    cf.coefficients = List.cons(head, tail)
        implies cf.denominator = continuant(tail)
} by {
    if cf.coefficients = List.cons(head, tail) {
        continued_fraction_denominator_eq_coefficients_denominator(cf)
        continued_fraction_denominator_cons_eq_tail_continuant(head, tail)
    }
}

/// A continued fraction with displayed nonempty coefficients has first
/// convergent projection equal to the displayed continuant.
theorem continued_fraction_cons_convergent_first_eq_continuant(
    cf: ContinuedFraction, head: Nat, tail: List[Nat]
) {
    cf.coefficients = List.cons(head, tail)
        implies cf.convergent.first = continuant(List.cons(head, tail))
} by {
    if cf.coefficients = List.cons(head, tail) {
        continued_fraction_convergent_eq_coefficients_convergent(cf)
        continued_fraction_convergent_cons_first_eq_continuant(head, tail)
    }
}

/// A continued fraction with displayed nonempty coefficients has second
/// convergent projection equal to the tail continuant.
theorem continued_fraction_cons_convergent_second_eq_tail_continuant(
    cf: ContinuedFraction, head: Nat, tail: List[Nat]
) {
    cf.coefficients = List.cons(head, tail)
        implies cf.convergent.second = continuant(tail)
} by {
    if cf.coefficients = List.cons(head, tail) {
        continued_fraction_convergent_eq_coefficients_convergent(cf)
        continued_fraction_convergent_cons_second_eq_tail_continuant(head, tail)
    }
}

/// A continued fraction with displayed nonempty coefficients has convergent
/// pair equal to the displayed continuant pair.
theorem continued_fraction_cons_convergent_eq_continuant_pair(
    cf: ContinuedFraction, head: Nat, tail: List[Nat]
) {
    cf.coefficients = List.cons(head, tail)
        implies cf.convergent =
            Pair.new(continuant(List.cons(head, tail)), continuant(tail))
} by {
    if cf.coefficients = List.cons(head, tail) {
        continued_fraction_convergent_eq_coefficients_convergent(cf)
        continued_fraction_convergent_cons_eq_continuant_pair(head, tail)
    }
}

/// A continued fraction with displayed nonempty coefficients has numerator
/// equal to its continuant method.
theorem continued_fraction_cons_numerator_eq_continuant_method(
    cf: ContinuedFraction, head: Nat, tail: List[Nat]
) {
    cf.coefficients = List.cons(head, tail)
        implies cf.numerator = cf.continuant
} by {
    if cf.coefficients = List.cons(head, tail) {
        continued_fraction_cons_numerator_eq_continuant(cf, head, tail)
        continued_fraction_continuant_eq_coefficients_continuant(cf)
    }
}

/// A continued fraction with displayed nonempty coefficients has first
/// convergent projection equal to its continuant method.
theorem continued_fraction_cons_convergent_first_eq_continuant_method(
    cf: ContinuedFraction, head: Nat, tail: List[Nat]
) {
    cf.coefficients = List.cons(head, tail)
        implies cf.convergent.first = cf.continuant
} by {
    if cf.coefficients = List.cons(head, tail) {
        continued_fraction_cons_convergent_first_eq_continuant(cf, head, tail)
        continued_fraction_continuant_eq_coefficients_continuant(cf)
    }
}

/// A valid coefficient list has numerator equal to its continuant.
theorem continued_fraction_numerator_eq_continuant_of_valid_coefficients(
    coefficients: List[Nat]
) {
    finite_continued_fraction_coefficients(coefficients)
        implies continued_fraction_numerator(coefficients) = continuant(coefficients)
} by {
    if finite_continued_fraction_coefficients(coefficients) {
        match coefficients {
            List.nil {
                finite_continued_fraction_coefficients(List.nil[Nat])
                false
            }
            List.cons(head, tail) {
                continued_fraction_numerator_cons_eq_continuant(head, tail)
                continued_fraction_numerator(List.cons(head, tail)) =
                    continuant(List.cons(head, tail))
                continued_fraction_numerator(coefficients) = continuant(coefficients)
            }
        }
    }
}

/// A valid coefficient list has convergent first projection equal to its
/// continuant.
theorem continued_fraction_convergent_first_eq_continuant_of_valid_coefficients(
    coefficients: List[Nat]
) {
    finite_continued_fraction_coefficients(coefficients)
        implies continued_fraction_convergent(coefficients).first = continuant(coefficients)
} by {
    if finite_continued_fraction_coefficients(coefficients) {
        continued_fraction_convergent_first_eq_numerator(coefficients)
        continued_fraction_numerator_eq_continuant_of_valid_coefficients(coefficients)
    }
}

/// A valid coefficient list has convergent pair formed by its continuant and
/// denominator.
theorem continued_fraction_convergent_eq_continuant_denominator_pair_of_valid_coefficients(
    coefficients: List[Nat]
) {
    finite_continued_fraction_coefficients(coefficients)
        implies continued_fraction_convergent(coefficients) =
            Pair.new(continuant(coefficients), continued_fraction_denominator(coefficients))
} by {
    if finite_continued_fraction_coefficients(coefficients) {
        continued_fraction_convergent_first_eq_continuant_of_valid_coefficients(coefficients)
        continued_fraction_convergent(coefficients).first = continuant(coefficients)
        continued_fraction_convergent_second_eq_denominator(coefficients)
        continued_fraction_convergent(coefficients).second =
            continued_fraction_denominator(coefficients)
        pair_new_first(continuant(coefficients), continued_fraction_denominator(coefficients))
        Pair.new(continuant(coefficients),
            continued_fraction_denominator(coefficients)).first = continuant(coefficients)
        pair_new_second(continuant(coefficients), continued_fraction_denominator(coefficients))
        Pair.new(continuant(coefficients),
            continued_fraction_denominator(coefficients)).second =
            continued_fraction_denominator(coefficients)
        pair_ext(continued_fraction_convergent(coefficients),
            Pair.new(continuant(coefficients), continued_fraction_denominator(coefficients)))
        continued_fraction_convergent(coefficients) =
            Pair.new(continuant(coefficients), continued_fraction_denominator(coefficients))
    }
}

/// The numerator method of a continued fraction is its continuant method.
theorem continued_fraction_numerator_eq_continuant_method(cf: ContinuedFraction) {
    cf.numerator = cf.continuant
} by {
    continued_fraction_coefficients_valid(cf)
    continued_fraction_numerator_eq_coefficients_numerator(cf)
    continued_fraction_continuant_eq_coefficients_continuant(cf)
    continued_fraction_numerator_eq_continuant_of_valid_coefficients(cf.coefficients)
}

/// The continuant method of a continued fraction is its numerator method.
theorem continued_fraction_continuant_eq_numerator_method(cf: ContinuedFraction) {
    cf.continuant = cf.numerator
} by {
    continued_fraction_numerator_eq_continuant_method(cf)
}

/// The first projection of the convergent method is the continuant method.
theorem continued_fraction_convergent_first_eq_continuant_method(
    cf: ContinuedFraction
) {
    cf.convergent.first = cf.continuant
} by {
    continued_fraction_convergent_first_eq_numerator_method(cf)
    continued_fraction_numerator_eq_continuant_method(cf)
}

/// The continuant method is the first projection of the convergent method.
theorem continued_fraction_continuant_method_eq_convergent_first(
    cf: ContinuedFraction
) {
    cf.continuant = cf.convergent.first
} by {
    continued_fraction_convergent_first_eq_continuant_method(cf)
}

/// The convergent method is the pair formed by the continuant and denominator
/// methods.
theorem continued_fraction_convergent_eq_continuant_denominator_method_pair(
    cf: ContinuedFraction
) {
    cf.convergent = Pair.new(cf.continuant, cf.denominator)
} by {
    continued_fraction_convergent_first_eq_continuant_method(cf)
    continued_fraction_convergent_second_eq_denominator_method(cf)
    pair_new_first(cf.continuant, cf.denominator)
    pair_new_second(cf.continuant, cf.denominator)
    pair_ext(cf.convergent, Pair.new(cf.continuant, cf.denominator))
}

/// A nonempty coefficient list whose whole list is positive has positive
/// numerator.
theorem continued_fraction_numerator_cons_positive_of_positive_tail(
    head: Nat, tail: List[Nat]
) {
    positive_continued_fraction_tail(List.cons(head, tail))
        implies Nat.0 < continued_fraction_numerator(List.cons(head, tail))
} by {
    if positive_continued_fraction_tail(List.cons(head, tail)) {
        continued_fraction_numerator_cons_eq_continuant(head, tail)
        continuant_positive_of_positive_tail(List.cons(head, tail))
        Nat.0 < continuant(List.cons(head, tail))
        Nat.0 < continued_fraction_numerator(List.cons(head, tail))
    }
}

/// A nonempty coefficient list whose whole list is positive has nonzero
/// numerator.
theorem continued_fraction_numerator_cons_ne_zero_of_positive_tail(
    head: Nat, tail: List[Nat]
) {
    positive_continued_fraction_tail(List.cons(head, tail))
        implies continued_fraction_numerator(List.cons(head, tail)) != Nat.0
} by {
    if positive_continued_fraction_tail(List.cons(head, tail)) {
        continued_fraction_numerator_cons_positive_of_positive_tail(head, tail)
        nat_positive_ne_zero(continued_fraction_numerator(List.cons(head, tail)))
    }
}

/// A nonempty coefficient list whose whole list is positive has positive
/// convergent first projection.
theorem continued_fraction_convergent_cons_first_positive_of_positive_tail(
    head: Nat, tail: List[Nat]
) {
    positive_continued_fraction_tail(List.cons(head, tail))
        implies Nat.0 < continued_fraction_convergent(List.cons(head, tail)).first
} by {
    if positive_continued_fraction_tail(List.cons(head, tail)) {
        continued_fraction_convergent_cons_first_eq_numerator(head, tail)
        continued_fraction_numerator_cons_positive_of_positive_tail(head, tail)
        Nat.0 < continued_fraction_numerator(List.cons(head, tail))
        Nat.0 < continued_fraction_convergent(List.cons(head, tail)).first
    }
}

/// A nonempty coefficient list whose whole list is positive has nonzero
/// convergent first projection.
theorem continued_fraction_convergent_cons_first_ne_zero_of_positive_tail(
    head: Nat, tail: List[Nat]
) {
    positive_continued_fraction_tail(List.cons(head, tail))
        implies continued_fraction_convergent(List.cons(head, tail)).first != Nat.0
} by {
    if positive_continued_fraction_tail(List.cons(head, tail)) {
        continued_fraction_convergent_cons_first_positive_of_positive_tail(head, tail)
        nat_positive_ne_zero(continued_fraction_convergent(List.cons(head, tail)).first)
    }
}

/// A continued fraction with displayed positive nonempty coefficients has
/// positive numerator method.
theorem continued_fraction_cons_numerator_positive_of_positive_tail(
    cf: ContinuedFraction, head: Nat, tail: List[Nat]
) {
    cf.coefficients = List.cons(head, tail)
        and positive_continued_fraction_tail(List.cons(head, tail))
        implies Nat.0 < cf.numerator
} by {
    if cf.coefficients = List.cons(head, tail)
        and positive_continued_fraction_tail(List.cons(head, tail)) {
        continued_fraction_cons_numerator_eq_continuant(cf, head, tail)
        cf.numerator = continuant(List.cons(head, tail))
        continuant_positive_of_positive_tail(List.cons(head, tail))
        Nat.0 < continuant(List.cons(head, tail))
        Nat.0 < cf.numerator
    }
}

/// A continued fraction with displayed positive nonempty coefficients has
/// nonzero numerator method.
theorem continued_fraction_cons_numerator_ne_zero_of_positive_tail(
    cf: ContinuedFraction, head: Nat, tail: List[Nat]
) {
    cf.coefficients = List.cons(head, tail)
        and positive_continued_fraction_tail(List.cons(head, tail))
        implies cf.numerator != Nat.0
} by {
    if cf.coefficients = List.cons(head, tail)
        and positive_continued_fraction_tail(List.cons(head, tail)) {
        continued_fraction_cons_numerator_positive_of_positive_tail(cf, head, tail)
        nat_positive_ne_zero(cf.numerator)
    }
}

/// A continued fraction with displayed positive nonempty coefficients has
/// positive continuant method.
theorem continued_fraction_cons_continuant_positive_of_positive_tail(
    cf: ContinuedFraction, head: Nat, tail: List[Nat]
) {
    cf.coefficients = List.cons(head, tail)
        and positive_continued_fraction_tail(List.cons(head, tail))
        implies Nat.0 < cf.continuant
} by {
    if cf.coefficients = List.cons(head, tail)
        and positive_continued_fraction_tail(List.cons(head, tail)) {
        continued_fraction_cons_numerator_positive_of_positive_tail(cf, head, tail)
        Nat.0 < cf.numerator
        continued_fraction_numerator_eq_continuant_method(cf)
        Nat.0 < cf.continuant
    }
}

/// A continued fraction with displayed positive nonempty coefficients has
/// nonzero continuant method.
theorem continued_fraction_cons_continuant_ne_zero_of_positive_tail(
    cf: ContinuedFraction, head: Nat, tail: List[Nat]
) {
    cf.coefficients = List.cons(head, tail)
        and positive_continued_fraction_tail(List.cons(head, tail))
        implies cf.continuant != Nat.0
} by {
    if cf.coefficients = List.cons(head, tail)
        and positive_continued_fraction_tail(List.cons(head, tail)) {
        continued_fraction_cons_continuant_positive_of_positive_tail(cf, head, tail)
        nat_positive_ne_zero(cf.continuant)
    }
}

/// A continued fraction with displayed positive nonempty coefficients has
/// positive convergent first projection.
theorem continued_fraction_cons_convergent_first_positive_of_positive_tail(
    cf: ContinuedFraction, head: Nat, tail: List[Nat]
) {
    cf.coefficients = List.cons(head, tail)
        and positive_continued_fraction_tail(List.cons(head, tail))
        implies Nat.0 < cf.convergent.first
} by {
    if cf.coefficients = List.cons(head, tail)
        and positive_continued_fraction_tail(List.cons(head, tail)) {
        continued_fraction_cons_convergent_first_eq_continuant(cf, head, tail)
        cf.convergent.first = continuant(List.cons(head, tail))
        continuant_positive_of_positive_tail(List.cons(head, tail))
        Nat.0 < continuant(List.cons(head, tail))
        Nat.0 < cf.convergent.first
    }
}

/// A continued fraction with displayed positive nonempty coefficients has
/// nonzero convergent first projection.
theorem continued_fraction_cons_convergent_first_ne_zero_of_positive_tail(
    cf: ContinuedFraction, head: Nat, tail: List[Nat]
) {
    cf.coefficients = List.cons(head, tail)
        and positive_continued_fraction_tail(List.cons(head, tail))
        implies cf.convergent.first != Nat.0
} by {
    if cf.coefficients = List.cons(head, tail)
        and positive_continued_fraction_tail(List.cons(head, tail)) {
        continued_fraction_cons_convergent_first_positive_of_positive_tail(
            cf, head, tail)
        nat_positive_ne_zero(cf.convergent.first)
    }
}

/// A valid coefficient list whose whole list is positive has positive
/// numerator.
theorem continued_fraction_numerator_positive_of_valid_positive_coefficients(
    coefficients: List[Nat]
) {
    finite_continued_fraction_coefficients(coefficients)
        and positive_continued_fraction_tail(coefficients)
        implies Nat.0 < continued_fraction_numerator(coefficients)
} by {
    if finite_continued_fraction_coefficients(coefficients)
        and positive_continued_fraction_tail(coefficients) {
        match coefficients {
            List.nil {
                finite_continued_fraction_coefficients(List.nil[Nat])
                false
            }
            List.cons(head, tail) {
                positive_continued_fraction_tail(List.cons(head, tail))
                continued_fraction_numerator_cons_positive_of_positive_tail(head, tail)
                Nat.0 < continued_fraction_numerator(List.cons(head, tail))
                Nat.0 < continued_fraction_numerator(coefficients)
            }
        }
    }
}

/// A valid coefficient list whose whole list is positive has nonzero
/// numerator.
theorem continued_fraction_numerator_ne_zero_of_valid_positive_coefficients(
    coefficients: List[Nat]
) {
    finite_continued_fraction_coefficients(coefficients)
        and positive_continued_fraction_tail(coefficients)
        implies continued_fraction_numerator(coefficients) != Nat.0
} by {
    if finite_continued_fraction_coefficients(coefficients)
        and positive_continued_fraction_tail(coefficients) {
        continued_fraction_numerator_positive_of_valid_positive_coefficients(coefficients)
        nat_positive_ne_zero(continued_fraction_numerator(coefficients))
    }
}

/// A valid coefficient list whose whole list is positive has positive
/// continuant.
theorem continuant_positive_of_valid_positive_coefficients(coefficients: List[Nat]) {
    finite_continued_fraction_coefficients(coefficients)
        and positive_continued_fraction_tail(coefficients)
        implies Nat.0 < continuant(coefficients)
} by {
    if finite_continued_fraction_coefficients(coefficients)
        and positive_continued_fraction_tail(coefficients) {
        continuant_positive_of_positive_tail(coefficients)
    }
}

/// A valid coefficient list whose whole list is positive has nonzero
/// continuant.
theorem continuant_ne_zero_of_valid_positive_coefficients(coefficients: List[Nat]) {
    finite_continued_fraction_coefficients(coefficients)
        and positive_continued_fraction_tail(coefficients)
        implies continuant(coefficients) != Nat.0
} by {
    if finite_continued_fraction_coefficients(coefficients)
        and positive_continued_fraction_tail(coefficients) {
        continuant_positive_of_valid_positive_coefficients(coefficients)
        nat_positive_ne_zero(continuant(coefficients))
    }
}

/// A valid coefficient list whose whole list is positive has positive
/// convergent first projection.
theorem continued_fraction_convergent_first_positive_of_valid_positive_coefficients(
    coefficients: List[Nat]
) {
    finite_continued_fraction_coefficients(coefficients)
        and positive_continued_fraction_tail(coefficients)
        implies Nat.0 < continued_fraction_convergent(coefficients).first
} by {
    if finite_continued_fraction_coefficients(coefficients)
        and positive_continued_fraction_tail(coefficients) {
        continued_fraction_convergent_first_eq_numerator(coefficients)
        continued_fraction_numerator_positive_of_valid_positive_coefficients(coefficients)
        Nat.0 < continued_fraction_numerator(coefficients)
        Nat.0 < continued_fraction_convergent(coefficients).first
    }
}

/// A valid coefficient list whose whole list is positive has nonzero
/// convergent first projection.
theorem continued_fraction_convergent_first_ne_zero_of_valid_positive_coefficients(
    coefficients: List[Nat]
) {
    finite_continued_fraction_coefficients(coefficients)
        and positive_continued_fraction_tail(coefficients)
        implies continued_fraction_convergent(coefficients).first != Nat.0
} by {
    if finite_continued_fraction_coefficients(coefficients)
        and positive_continued_fraction_tail(coefficients) {
        continued_fraction_convergent_first_positive_of_valid_positive_coefficients(
            coefficients)
        nat_positive_ne_zero(continued_fraction_convergent(coefficients).first)
    }
}

/// A continued fraction whose whole coefficient list is positive has positive
/// numerator method.
theorem continued_fraction_numerator_positive_of_positive_coefficients(
    cf: ContinuedFraction
) {
    positive_continued_fraction_tail(cf.coefficients) implies Nat.0 < cf.numerator
} by {
    if positive_continued_fraction_tail(cf.coefficients) {
        continued_fraction_coefficients_valid(cf)
        continued_fraction_numerator_eq_coefficients_numerator(cf)
        continued_fraction_numerator_positive_of_valid_positive_coefficients(
            cf.coefficients)
        Nat.0 < continued_fraction_numerator(cf.coefficients)
        Nat.0 < cf.numerator
    }
}

/// A continued fraction whose whole coefficient list is positive has nonzero
/// numerator method.
theorem continued_fraction_numerator_ne_zero_of_positive_coefficients(
    cf: ContinuedFraction
) {
    positive_continued_fraction_tail(cf.coefficients) implies cf.numerator != Nat.0
} by {
    if positive_continued_fraction_tail(cf.coefficients) {
        continued_fraction_numerator_positive_of_positive_coefficients(cf)
        nat_positive_ne_zero(cf.numerator)
    }
}

/// A continued fraction whose whole coefficient list is positive has positive
/// convergent first projection.
theorem continued_fraction_convergent_first_positive_of_positive_coefficients(
    cf: ContinuedFraction
) {
    positive_continued_fraction_tail(cf.coefficients)
        implies Nat.0 < cf.convergent.first
} by {
    if positive_continued_fraction_tail(cf.coefficients) {
        continued_fraction_coefficients_valid(cf)
        continued_fraction_convergent_eq_coefficients_convergent(cf)
        continued_fraction_convergent_first_positive_of_valid_positive_coefficients(
            cf.coefficients)
        Nat.0 < continued_fraction_convergent(cf.coefficients).first
        Nat.0 < cf.convergent.first
    }
}

/// A continued fraction whose whole coefficient list is positive has nonzero
/// convergent first projection.
theorem continued_fraction_convergent_first_ne_zero_of_positive_coefficients(
    cf: ContinuedFraction
) {
    positive_continued_fraction_tail(cf.coefficients)
        implies cf.convergent.first != Nat.0
} by {
    if positive_continued_fraction_tail(cf.coefficients) {
        continued_fraction_convergent_first_positive_of_positive_coefficients(cf)
        nat_positive_ne_zero(cf.convergent.first)
    }
}

/// A singleton convergent state is the pair of next recurrence states.
theorem continued_fraction_convergent_state_singleton_pair(head: Nat,
    pn: Nat, cn: Nat, pd: Nat, cd: Nat) {
    continued_fraction_convergent_state(List.cons(head, List.nil[Nat]),
        pn, cn, pd, cd) =
        Pair.new(cn * head + pn, cd * head + pd)
} by {
    continued_fraction_convergent_state_singleton_first(head, pn, cn, pd, cd)
    continued_fraction_convergent_state_singleton_second(head, pn, cn, pd, cd)
    pair_new_first(cn * head + pn, cd * head + pd)
    pair_new_second(cn * head + pn, cd * head + pd)
    pair_ext(continued_fraction_convergent_state(List.cons(head, List.nil[Nat]),
        pn, cn, pd, cd), Pair.new(cn * head + pn, cd * head + pd))
}

/// A two-coefficient convergent state is the pair of recurrence states after
/// two steps.
theorem continued_fraction_convergent_state_pair_pair(head: Nat, next: Nat,
    pn: Nat, cn: Nat, pd: Nat, cd: Nat) {
    continued_fraction_convergent_state(List.cons(head, List.cons(next, List.nil[Nat])),
        pn, cn, pd, cd) =
        Pair.new((cn * head + pn) * next + cn,
            (cd * head + pd) * next + cd)
} by {
    continued_fraction_convergent_state_pair_first(head, next, pn, cn, pd, cd)
    continued_fraction_convergent_state_pair_second(head, next, pn, cn, pd, cd)
    pair_new_first((cn * head + pn) * next + cn,
        (cd * head + pd) * next + cd)
    pair_new_second((cn * head + pn) * next + cn,
        (cd * head + pd) * next + cd)
    pair_ext(continued_fraction_convergent_state(
        List.cons(head, List.cons(next, List.nil[Nat])), pn, cn, pd, cd),
        Pair.new((cn * head + pn) * next + cn,
            (cd * head + pd) * next + cd))
}

/// A three-coefficient convergent state is the pair of recurrence states after
/// three steps.
theorem continued_fraction_convergent_state_triple_pair(head: Nat, next: Nat,
    third: Nat, pn: Nat, cn: Nat, pd: Nat, cd: Nat) {
    continued_fraction_convergent_state(
        List.cons(head, List.cons(next, List.cons(third, List.nil[Nat]))),
        pn, cn, pd, cd) =
        Pair.new(((cn * head + pn) * next + cn) * third + (cn * head + pn),
            ((cd * head + pd) * next + cd) * third + (cd * head + pd))
} by {
    continued_fraction_convergent_state_triple_first(head, next, third,
        pn, cn, pd, cd)
    continued_fraction_convergent_state_triple_second(head, next, third,
        pn, cn, pd, cd)
    pair_new_first(((cn * head + pn) * next + cn) * third + (cn * head + pn),
        ((cd * head + pd) * next + cd) * third + (cd * head + pd))
    pair_new_second(((cn * head + pn) * next + cn) * third + (cn * head + pn),
        ((cd * head + pd) * next + cd) * third + (cd * head + pd))
    pair_ext(continued_fraction_convergent_state(
        List.cons(head, List.cons(next, List.cons(third, List.nil[Nat]))),
        pn, cn, pd, cd),
        Pair.new(((cn * head + pn) * next + cn) * third + (cn * head + pn),
            ((cd * head + pd) * next + cd) * third + (cd * head + pd)))
}

/// The singleton convergent has numerator equal to its coefficient.
theorem continued_fraction_convergent_singleton_first(head: Nat) {
    continued_fraction_convergent(List.cons(head, List.nil[Nat])).first = head
} by {
    continued_fraction_convergent(List.cons(head, List.nil[Nat])) =
        continued_fraction_convergent_state(List.cons(head, List.nil[Nat]), Nat.0,
            Nat.1, Nat.1, Nat.0)
    continued_fraction_convergent_state_cons(head, List.nil[Nat], Nat.0, Nat.1, Nat.1, Nat.0)
    continued_fraction_convergent_state_nil(Nat.1, Nat.1 * head + Nat.0, Nat.0,
        Nat.0 * head + Nat.1)
    continued_fraction_convergent_state(List.cons(head, List.nil[Nat]), Nat.0,
        Nat.1, Nat.1, Nat.0) =
        Pair.new(Nat.1 * head + Nat.0, Nat.0 * head + Nat.1)
    pair_new_first(Nat.1 * head + Nat.0, Nat.0 * head + Nat.1)
    mul_one_left(head)
    Nat.1 * head = head
    Nat.1 * head + Nat.0 = head
    Pair.new(Nat.1 * head + Nat.0, Nat.0 * head + Nat.1).first = head
    continued_fraction_convergent(List.cons(head, List.nil[Nat])).first = head
}

/// The singleton convergent has denominator one.
theorem continued_fraction_convergent_singleton_second(head: Nat) {
    continued_fraction_convergent(List.cons(head, List.nil[Nat])).second = Nat.1
} by {
    continued_fraction_convergent(List.cons(head, List.nil[Nat])) =
        continued_fraction_convergent_state(List.cons(head, List.nil[Nat]), Nat.0,
            Nat.1, Nat.1, Nat.0)
    continued_fraction_convergent_state_cons(head, List.nil[Nat], Nat.0, Nat.1, Nat.1, Nat.0)
    continued_fraction_convergent_state_nil(Nat.1, Nat.1 * head + Nat.0, Nat.0,
        Nat.0 * head + Nat.1)
    continued_fraction_convergent_state(List.cons(head, List.nil[Nat]), Nat.0,
        Nat.1, Nat.1, Nat.0) =
        Pair.new(Nat.1 * head + Nat.0, Nat.0 * head + Nat.1)
    pair_new_second(Nat.1 * head + Nat.0, Nat.0 * head + Nat.1)
    mul_zero_right(head)
    Nat.0 * head = Nat.0
    Nat.0 * head + Nat.1 = Nat.1
    Pair.new(Nat.1 * head + Nat.0, Nat.0 * head + Nat.1).second = Nat.1
    continued_fraction_convergent(List.cons(head, List.nil[Nat])).second = Nat.1
}

/// A singleton continued fraction has denominator one.
theorem continued_fraction_denominator_singleton(head: Nat) {
    continued_fraction_denominator(List.cons(head, List.nil[Nat])) = Nat.1
}

/// A singleton continued fraction has denominator one.
theorem continued_fraction_singleton_denominator(head: Nat) {
    ContinuedFraction.singleton(head).denominator = Nat.1
} by {
    continued_fraction_singleton_coefficients(head)
    continued_fraction_denominator_eq_coefficients_denominator(ContinuedFraction.singleton(head))
    continued_fraction_denominator_singleton(head)
}

/// The singleton convergent first projection matches the numerator definition.
theorem continued_fraction_convergent_singleton_first_eq_numerator(head: Nat) {
    continued_fraction_convergent(List.cons(head, List.nil[Nat])).first =
        continued_fraction_numerator(List.cons(head, List.nil[Nat]))
} by {
    continued_fraction_convergent_singleton_first(head)
    continued_fraction_numerator_singleton(head)
}

/// The singleton convergent second projection matches the denominator definition.
theorem continued_fraction_convergent_singleton_second_eq_denominator(head: Nat) {
    continued_fraction_convergent(List.cons(head, List.nil[Nat])).second =
        continued_fraction_denominator(List.cons(head, List.nil[Nat]))
} by {
    continued_fraction_convergent_singleton_second(head)
    continued_fraction_denominator_singleton(head)
}

/// The singleton convergent has positive denominator.
theorem continued_fraction_convergent_singleton_second_positive(head: Nat) {
    Nat.0 < continued_fraction_convergent(List.cons(head, List.nil[Nat])).second
} by {
    continued_fraction_convergent_singleton_second(head)
    Nat.0 < Nat.1
}

/// The singleton convergent has nonzero denominator.
theorem continued_fraction_convergent_singleton_second_ne_zero(head: Nat) {
    continued_fraction_convergent(List.cons(head, List.nil[Nat])).second != Nat.0
} by {
    continued_fraction_convergent_singleton_second_positive(head)
    nat_positive_ne_zero(continued_fraction_convergent(List.cons(head, List.nil[Nat])).second)
}

/// A singleton continued fraction has convergent numerator equal to its coefficient.
theorem continued_fraction_singleton_convergent_first(head: Nat) {
    ContinuedFraction.singleton(head).convergent.first = head
} by {
    continued_fraction_singleton_coefficients(head)
    continued_fraction_convergent_eq_coefficients_convergent(ContinuedFraction.singleton(head))
    continued_fraction_convergent_singleton_first(head)
}

/// A singleton continued fraction has convergent denominator one.
theorem continued_fraction_singleton_convergent_second(head: Nat) {
    ContinuedFraction.singleton(head).convergent.second = Nat.1
} by {
    continued_fraction_singleton_coefficients(head)
    continued_fraction_convergent_eq_coefficients_convergent(ContinuedFraction.singleton(head))
    continued_fraction_convergent_singleton_second(head)
}

/// A two-term continued fraction has denominator equal to its tail coefficient.
theorem continued_fraction_denominator_pair(head: Nat, next: Nat) {
    continued_fraction_denominator(List.cons(head, List.cons(next, List.nil[Nat]))) =
        next
}

/// The two-term convergent has numerator `head * next + 1`.
theorem continued_fraction_convergent_pair_first(head: Nat, next: Nat) {
    continued_fraction_convergent(List.cons(head, List.cons(next, List.nil[Nat]))).first =
        head * next + Nat.1
} by {
    continued_fraction_convergent(List.cons(head, List.cons(next, List.nil[Nat]))) =
        continued_fraction_convergent_state(List.cons(head, List.cons(next, List.nil[Nat])),
            Nat.0, Nat.1, Nat.1, Nat.0)
    continued_fraction_convergent_state_cons(head, List.cons(next, List.nil[Nat]),
        Nat.0, Nat.1, Nat.1, Nat.0)
    continued_fraction_convergent_state_cons(next, List.nil[Nat],
        Nat.1, Nat.1 * head + Nat.0, Nat.0, Nat.0 * head + Nat.1)
    continued_fraction_convergent_state_nil(Nat.1 * head + Nat.0,
        (Nat.1 * head + Nat.0) * next + Nat.1,
        Nat.0 * head + Nat.1,
        (Nat.0 * head + Nat.1) * next + Nat.0)
    continued_fraction_convergent_state(List.cons(head, List.cons(next, List.nil[Nat])),
        Nat.0, Nat.1, Nat.1, Nat.0) =
        Pair.new((Nat.1 * head + Nat.0) * next + Nat.1,
            (Nat.0 * head + Nat.1) * next + Nat.0)
    pair_new_first((Nat.1 * head + Nat.0) * next + Nat.1,
        (Nat.0 * head + Nat.1) * next + Nat.0)
    mul_one_left(head)
    Nat.1 * head = head
    Nat.1 * head + Nat.0 = head
    (Nat.1 * head + Nat.0) * next = head * next
    (Nat.1 * head + Nat.0) * next + Nat.1 = head * next + Nat.1
    Pair.new((Nat.1 * head + Nat.0) * next + Nat.1,
        (Nat.0 * head + Nat.1) * next + Nat.0).first = head * next + Nat.1
    continued_fraction_convergent(List.cons(head, List.cons(next, List.nil[Nat]))).first =
        head * next + Nat.1
}

/// The two-term convergent has denominator equal to the second coefficient.
theorem continued_fraction_convergent_pair_second(head: Nat, next: Nat) {
    continued_fraction_convergent(List.cons(head, List.cons(next, List.nil[Nat]))).second =
        next
} by {
    continued_fraction_convergent(List.cons(head, List.cons(next, List.nil[Nat]))) =
        continued_fraction_convergent_state(List.cons(head, List.cons(next, List.nil[Nat])),
            Nat.0, Nat.1, Nat.1, Nat.0)
    continued_fraction_convergent_state_cons(head, List.cons(next, List.nil[Nat]),
        Nat.0, Nat.1, Nat.1, Nat.0)
    continued_fraction_convergent_state_cons(next, List.nil[Nat],
        Nat.1, Nat.1 * head + Nat.0, Nat.0, Nat.0 * head + Nat.1)
    continued_fraction_convergent_state_nil(Nat.1 * head + Nat.0,
        (Nat.1 * head + Nat.0) * next + Nat.1,
        Nat.0 * head + Nat.1,
        (Nat.0 * head + Nat.1) * next + Nat.0)
    continued_fraction_convergent_state(List.cons(head, List.cons(next, List.nil[Nat])),
        Nat.0, Nat.1, Nat.1, Nat.0) =
        Pair.new((Nat.1 * head + Nat.0) * next + Nat.1,
            (Nat.0 * head + Nat.1) * next + Nat.0)
    pair_new_second((Nat.1 * head + Nat.0) * next + Nat.1,
        (Nat.0 * head + Nat.1) * next + Nat.0)
    mul_zero_right(head)
    Nat.0 * head = Nat.0
    Nat.0 * head + Nat.1 = Nat.1
    mul_one_left(next)
    Nat.1 * next = next
    (Nat.0 * head + Nat.1) * next = next
    (Nat.0 * head + Nat.1) * next + Nat.0 = next
    Pair.new((Nat.1 * head + Nat.0) * next + Nat.1,
        (Nat.0 * head + Nat.1) * next + Nat.0).second = next
    continued_fraction_convergent(List.cons(head, List.cons(next, List.nil[Nat]))).second =
        next
}

/// The two-term convergent first projection matches the numerator definition.
theorem continued_fraction_convergent_pair_first_eq_numerator(head: Nat, next: Nat) {
    continued_fraction_convergent(List.cons(head, List.cons(next, List.nil[Nat]))).first =
        continued_fraction_numerator(List.cons(head, List.cons(next, List.nil[Nat])))
} by {
    continued_fraction_convergent_pair_first(head, next)
    continued_fraction_numerator_pair(head, next)
}

/// The two-term convergent second projection matches the denominator definition.
theorem continued_fraction_convergent_pair_second_eq_denominator(head: Nat, next: Nat) {
    continued_fraction_convergent(List.cons(head, List.cons(next, List.nil[Nat]))).second =
        continued_fraction_denominator(List.cons(head, List.cons(next, List.nil[Nat])))
} by {
    continued_fraction_convergent_pair_second(head, next)
    continued_fraction_denominator_pair(head, next)
}

/// A two-term convergent with positive second coefficient has positive denominator.
theorem continued_fraction_convergent_pair_second_positive_of_tail_positive(head: Nat,
    next: Nat) {
    Nat.0 < next implies
        Nat.0 < continued_fraction_convergent(
            List.cons(head, List.cons(next, List.nil[Nat]))).second
} by {
    if Nat.0 < next {
        continued_fraction_convergent_pair_second(head, next)
    }
}

/// A valid two-term convergent has positive denominator.
theorem continued_fraction_convergent_pair_second_positive(head: Nat, next: Nat) {
    finite_continued_fraction_coefficients(List.cons(head, List.cons(next, List.nil[Nat])))
        implies Nat.0 < continued_fraction_convergent(
            List.cons(head, List.cons(next, List.nil[Nat]))).second
} by {
    if finite_continued_fraction_coefficients(List.cons(head, List.cons(next, List.nil[Nat]))) {
        finite_continued_fraction_coefficients_pair_tail_positive(head, next)
        continued_fraction_convergent_pair_second_positive_of_tail_positive(head, next)
    }
}

/// A valid two-term convergent has nonzero denominator.
theorem continued_fraction_convergent_pair_second_ne_zero(head: Nat, next: Nat) {
    finite_continued_fraction_coefficients(List.cons(head, List.cons(next, List.nil[Nat])))
        implies continued_fraction_convergent(
            List.cons(head, List.cons(next, List.nil[Nat]))).second != Nat.0
} by {
    if finite_continued_fraction_coefficients(List.cons(head, List.cons(next, List.nil[Nat]))) {
        continued_fraction_convergent_pair_second_positive(head, next)
        nat_positive_ne_zero(continued_fraction_convergent(
            List.cons(head, List.cons(next, List.nil[Nat]))).second)
    }
}

/// A three-term continued fraction has denominator equal to the tail continuant.
theorem continued_fraction_denominator_triple(head: Nat, next: Nat, third: Nat) {
    continued_fraction_denominator(
        List.cons(head, List.cons(next, List.cons(third, List.nil[Nat])))) =
        next * third + Nat.1
} by {
    continuant_pair(next, third)
}

/// A singleton continued fraction has positive denominator.
theorem continued_fraction_denominator_singleton_positive(head: Nat) {
    Nat.0 < continued_fraction_denominator(List.cons(head, List.nil[Nat]))
} by {
    continued_fraction_denominator_singleton(head)
    continued_fraction_denominator(List.cons(head, List.nil[Nat])) = Nat.1
    Nat.0 < Nat.1
}

/// A valid two-term continued fraction has positive denominator.
theorem continued_fraction_denominator_pair_positive(head: Nat, next: Nat) {
    finite_continued_fraction_coefficients(List.cons(head, List.cons(next, List.nil[Nat])))
        implies Nat.0 < continued_fraction_denominator(List.cons(head, List.cons(next, List.nil[Nat])))
} by {
    if finite_continued_fraction_coefficients(List.cons(head, List.cons(next, List.nil[Nat]))) {
        finite_continued_fraction_coefficients_pair_tail_positive(head, next)
        Nat.0 < next
        continued_fraction_denominator_pair(head, next)
        continued_fraction_denominator(List.cons(head, List.cons(next, List.nil[Nat]))) =
            next
    }
}

/// A valid two-term continued fraction has nonzero denominator.
theorem continued_fraction_denominator_pair_ne_zero(head: Nat, next: Nat) {
    finite_continued_fraction_coefficients(List.cons(head, List.cons(next, List.nil[Nat])))
        implies continued_fraction_denominator(List.cons(head, List.cons(next, List.nil[Nat]))) != Nat.0
} by {
    if finite_continued_fraction_coefficients(List.cons(head, List.cons(next, List.nil[Nat]))) {
        continued_fraction_denominator_pair_positive(head, next)
        nat_positive_ne_zero(continued_fraction_denominator(
            List.cons(head, List.cons(next, List.nil[Nat]))))
    }
}

/// A two-term continued fraction has denominator equal to the second coefficient.
theorem continued_fraction_pair_denominator(cf: ContinuedFraction, head: Nat, next: Nat) {
    cf.coefficients = List.cons(head, List.cons(next, List.nil[Nat]))
        implies cf.denominator = next
} by {
    if cf.coefficients = List.cons(head, List.cons(next, List.nil[Nat])) {
        continued_fraction_denominator_eq_coefficients_denominator(cf)
        continued_fraction_denominator_pair(head, next)
    }
}

/// A two-term continued fraction has positive denominator.
theorem continued_fraction_pair_denominator_positive(cf: ContinuedFraction,
    head: Nat, next: Nat) {
    cf.coefficients = List.cons(head, List.cons(next, List.nil[Nat]))
        implies Nat.0 < cf.denominator
} by {
    if cf.coefficients = List.cons(head, List.cons(next, List.nil[Nat])) {
        continued_fraction_coefficients_valid(cf)
        finite_continued_fraction_coefficients(cf.coefficients)
        finite_continued_fraction_coefficients(List.cons(head, List.cons(next, List.nil[Nat])))
        finite_continued_fraction_coefficients_pair_tail_positive(head, next)
        continued_fraction_pair_denominator(cf, head, next)
        Nat.0 < next
        cf.denominator = next
        Nat.0 < cf.denominator
    }
}

/// A three-term continued fraction with positive tail coefficients has positive denominator.
theorem continued_fraction_denominator_triple_positive_of_tail_positive(head: Nat,
    next: Nat, third: Nat) {
    Nat.0 < next and Nat.0 < third implies
        Nat.0 < continued_fraction_denominator(
            List.cons(head, List.cons(next, List.cons(third, List.nil[Nat]))))
} by {
    if Nat.0 < next and Nat.0 < third {
        continued_fraction_denominator_triple(head, next, third)
        nat_mul_positive(next, third)
        Nat.0 < next * third
        nat_add_positive_left(next * third, Nat.1)
        Nat.0 < next * third + Nat.1
    }
}

/// A valid three-term continued fraction has positive denominator.
theorem continued_fraction_denominator_triple_positive(head: Nat, next: Nat,
    third: Nat) {
    finite_continued_fraction_coefficients(
        List.cons(head, List.cons(next, List.cons(third, List.nil[Nat]))))
        implies Nat.0 < continued_fraction_denominator(
            List.cons(head, List.cons(next, List.cons(third, List.nil[Nat]))))
} by {
    if finite_continued_fraction_coefficients(
        List.cons(head, List.cons(next, List.cons(third, List.nil[Nat])))) {
        finite_continued_fraction_coefficients_cons_tail(head,
            List.cons(next, List.cons(third, List.nil[Nat])))
        positive_continued_fraction_tail(List.cons(next, List.cons(third, List.nil[Nat])))
        positive_continued_fraction_tail_cons_head(next, List.cons(third, List.nil[Nat]))
        positive_continued_fraction_tail_cons_tail(next, List.cons(third, List.nil[Nat]))
        positive_continued_fraction_tail(List.cons(third, List.nil[Nat]))
        positive_continued_fraction_tail_cons_head(third, List.nil[Nat])
        Nat.0 < next
        Nat.0 < third
        continued_fraction_denominator_triple_positive_of_tail_positive(head, next, third)
        Nat.0 < continued_fraction_denominator(
            List.cons(head, List.cons(next, List.cons(third, List.nil[Nat]))))
    }
}

/// A valid three-term continued fraction has nonzero denominator.
theorem continued_fraction_denominator_triple_ne_zero(head: Nat, next: Nat,
    third: Nat) {
    finite_continued_fraction_coefficients(
        List.cons(head, List.cons(next, List.cons(third, List.nil[Nat]))))
        implies continued_fraction_denominator(
            List.cons(head, List.cons(next, List.cons(third, List.nil[Nat])))) != Nat.0
} by {
    if finite_continued_fraction_coefficients(
        List.cons(head, List.cons(next, List.cons(third, List.nil[Nat])))) {
        continued_fraction_denominator_triple_positive(head, next, third)
        nat_positive_ne_zero(continued_fraction_denominator(
            List.cons(head, List.cons(next, List.cons(third, List.nil[Nat])))))
    }
}

/// A three-term continued fraction has denominator equal to the tail continuant.
theorem continued_fraction_triple_denominator(cf: ContinuedFraction, head: Nat,
    next: Nat, third: Nat) {
    cf.coefficients = List.cons(head, List.cons(next, List.cons(third, List.nil[Nat])))
        implies cf.denominator = next * third + Nat.1
} by {
    if cf.coefficients = List.cons(head, List.cons(next, List.cons(third, List.nil[Nat]))) {
        continued_fraction_denominator_eq_coefficients_denominator(cf)
        continued_fraction_denominator_triple(head, next, third)
    }
}

/// A three-term continued fraction has positive denominator.
theorem continued_fraction_triple_denominator_positive(cf: ContinuedFraction,
    head: Nat, next: Nat, third: Nat) {
    cf.coefficients = List.cons(head, List.cons(next, List.cons(third, List.nil[Nat])))
        implies Nat.0 < cf.denominator
} by {
    if cf.coefficients = List.cons(head, List.cons(next, List.cons(third, List.nil[Nat]))) {
        continued_fraction_coefficients_valid(cf)
        finite_continued_fraction_coefficients(cf.coefficients)
        finite_continued_fraction_coefficients(
            List.cons(head, List.cons(next, List.cons(third, List.nil[Nat]))))
        continued_fraction_denominator_triple_positive(head, next, third)
        continued_fraction_denominator_eq_coefficients_denominator(cf)
        Nat.0 < continued_fraction_denominator(
            List.cons(head, List.cons(next, List.cons(third, List.nil[Nat]))))
        Nat.0 < cf.denominator
    }
}

/// The singleton numerator is the coefficient times the denominator.
theorem continued_fraction_numerator_singleton_recurrence(head: Nat) {
    continued_fraction_numerator(List.cons(head, List.nil[Nat])) =
        head * continued_fraction_denominator(List.cons(head, List.nil[Nat]))
} by {
    continued_fraction_numerator_singleton(head)
    continued_fraction_denominator_singleton(head)
    head * continued_fraction_denominator(List.cons(head, List.nil[Nat])) =
        head * Nat.1
    mul_one_right(head)
}

/// The two-term numerator satisfies the first nontrivial continuant recurrence.
theorem continued_fraction_numerator_pair_recurrence(head: Nat, next: Nat) {
    continued_fraction_numerator(List.cons(head, List.cons(next, List.nil[Nat]))) =
        head * continued_fraction_denominator(List.cons(head, List.cons(next, List.nil[Nat]))) +
            Nat.1
} by {
    continued_fraction_numerator_pair(head, next)
    continued_fraction_denominator_pair(head, next)
    head * continued_fraction_denominator(List.cons(head, List.cons(next, List.nil[Nat]))) =
        head * next
}

/// The value of a two-term continued fraction is `head + 1 / next`.
theorem continued_fraction_value_pair(head: Nat, next: Nat) {
    continued_fraction_value(List.cons(head, List.cons(next, List.nil[Nat]))) =
        Rat.from_nat(head) + Rat.from_nat(next).inverse
} by {
    continued_fraction_value_singleton(next)
}

/// The value of a two-term continued fraction is `head + 1 / next`.
theorem continued_fraction_pair_value(cf: ContinuedFraction, head: Nat, next: Nat) {
    cf.coefficients = List.cons(head, List.cons(next, List.nil[Nat]))
        implies cf.value = Rat.from_nat(head) + Rat.from_nat(next).inverse
} by {
    if cf.coefficients = List.cons(head, List.cons(next, List.nil[Nat])) {
        continued_fraction_value_eq_coefficients_value(cf)
        continued_fraction_value_pair(head, next)
    }
}

/// The value of a three-term continued fraction unfolds through the tail pair.
theorem continued_fraction_value_triple(head: Nat, next: Nat, third: Nat) {
    continued_fraction_value(List.cons(head, List.cons(next, List.cons(third, List.nil[Nat])))) =
        Rat.from_nat(head) +
            (Rat.from_nat(next) + Rat.from_nat(third).inverse).inverse
} by {
    continued_fraction_value_pair(next, third)
}

/// The value of a three-term continued fraction unfolds through the tail pair.
theorem continued_fraction_triple_value(cf: ContinuedFraction, head: Nat, next: Nat,
    third: Nat) {
    cf.coefficients = List.cons(head, List.cons(next, List.cons(third, List.nil[Nat])))
        implies cf.value =
            Rat.from_nat(head) +
                (Rat.from_nat(next) + Rat.from_nat(third).inverse).inverse
} by {
    if cf.coefficients = List.cons(head, List.cons(next, List.cons(third, List.nil[Nat]))) {
        continued_fraction_value_eq_coefficients_value(cf)
        continued_fraction_value_triple(head, next, third)
    }
}

/// Adding two singleton continued-fraction values adds their coefficients.
theorem continued_fraction_value_singleton_add(left: Nat, right: Nat) {
    continued_fraction_value(List.cons(left, List.nil[Nat])) +
        continued_fraction_value(List.cons(right, List.nil[Nat])) =
        Rat.from_nat(left + right)
} by {
    continued_fraction_value_singleton(left)
    continued_fraction_value_singleton(right)
    from_nat_add(left, right)
}

/// Adding two singleton continued-fraction values adds their coefficients.
theorem continued_fraction_singleton_value_add(left: Nat, right: Nat) {
    ContinuedFraction.singleton(left).value + ContinuedFraction.singleton(right).value =
        Rat.from_nat(left + right)
} by {
    continued_fraction_singleton_value(left)
    continued_fraction_singleton_value(right)
    from_nat_add(left, right)
}

/// The singleton convergent is the pair formed by its numerator and
/// denominator.
theorem continued_fraction_convergent_singleton_eq_pair(head: Nat) {
    continued_fraction_convergent(List.cons(head, List.nil[Nat])) =
        Pair.new(head, Nat.1)
} by {
    continued_fraction_convergent_singleton_first(head)
    continued_fraction_convergent_singleton_second(head)
    pair_new_first(head, Nat.1)
    pair_new_second(head, Nat.1)
    pair_ext(continued_fraction_convergent(List.cons(head, List.nil[Nat])),
        Pair.new(head, Nat.1))
}

/// The two-term convergent is the pair formed by its numerator and denominator.
theorem continued_fraction_convergent_pair_eq_pair(head: Nat, next: Nat) {
    continued_fraction_convergent(List.cons(head, List.cons(next, List.nil[Nat]))) =
        Pair.new(head * next + Nat.1, next)
} by {
    continued_fraction_convergent_pair_first(head, next)
    continued_fraction_convergent_pair_second(head, next)
    pair_new_first(head * next + Nat.1, next)
    pair_new_second(head * next + Nat.1, next)
    pair_ext(continued_fraction_convergent(
        List.cons(head, List.cons(next, List.nil[Nat]))),
        Pair.new(head * next + Nat.1, next))
}

/// The three-term convergent first projection matches the numerator.
theorem continued_fraction_convergent_triple_first_eq_numerator(
    head: Nat, next: Nat, third: Nat
) {
    continued_fraction_convergent(
        List.cons(head, List.cons(next, List.cons(third, List.nil[Nat])))).first =
        continued_fraction_numerator(
            List.cons(head, List.cons(next, List.cons(third, List.nil[Nat]))))
} by {
    continued_fraction_convergent_state_first_projection_at_triple(head, next, third,
        Nat.0, Nat.1, Nat.1, Nat.0)
    continued_fraction_convergent_cons_first_eq_numerator_of_projection_at(head,
        List.cons(next, List.cons(third, List.nil[Nat])))
}

/// The three-term convergent has numerator given by the third continuant.
theorem continued_fraction_convergent_triple_first(head: Nat, next: Nat, third: Nat) {
    continued_fraction_convergent(
        List.cons(head, List.cons(next, List.cons(third, List.nil[Nat])))).first =
        (head * next + Nat.1) * third + head
} by {
    continued_fraction_convergent_triple_first_eq_numerator(head, next, third)
    continued_fraction_numerator_triple(head, next, third)
}

/// The three-term convergent second projection matches the denominator.
theorem continued_fraction_convergent_triple_second_eq_denominator(
    head: Nat, next: Nat, third: Nat
) {
    continued_fraction_convergent(
        List.cons(head, List.cons(next, List.cons(third, List.nil[Nat])))).second =
        continued_fraction_denominator(
            List.cons(head, List.cons(next, List.cons(third, List.nil[Nat]))))
} by {
    continued_fraction_convergent_state_second_projection_at_triple(head, next, third,
        Nat.0, Nat.1, Nat.1, Nat.0)
    continued_fraction_convergent_cons_second_eq_denominator_of_projection_at(head,
        List.cons(next, List.cons(third, List.nil[Nat])))
}

/// The three-term convergent has denominator equal to the tail continuant.
theorem continued_fraction_convergent_triple_second(head: Nat, next: Nat, third: Nat) {
    continued_fraction_convergent(
        List.cons(head, List.cons(next, List.cons(third, List.nil[Nat])))).second =
        next * third + Nat.1
} by {
    continued_fraction_convergent_triple_second_eq_denominator(head, next, third)
    continued_fraction_denominator_triple(head, next, third)
}

/// The three-term convergent is the pair formed by its numerator and
/// denominator.
theorem continued_fraction_convergent_triple_eq_pair(
    head: Nat, next: Nat, third: Nat
) {
    continued_fraction_convergent(
        List.cons(head, List.cons(next, List.cons(third, List.nil[Nat])))) =
        Pair.new((head * next + Nat.1) * third + head, next * third + Nat.1)
} by {
    continued_fraction_convergent_triple_first(head, next, third)
    continued_fraction_convergent_triple_second(head, next, third)
    pair_new_first((head * next + Nat.1) * third + head, next * third + Nat.1)
    pair_new_second((head * next + Nat.1) * third + head, next * third + Nat.1)
    pair_ext(continued_fraction_convergent(
        List.cons(head, List.cons(next, List.cons(third, List.nil[Nat])))),
        Pair.new((head * next + Nat.1) * third + head, next * third + Nat.1))
}

/// A three-term convergent with positive tail coefficients has positive
/// denominator projection.
theorem continued_fraction_convergent_triple_second_positive_of_tail_positive(
    head: Nat, next: Nat, third: Nat
) {
    Nat.0 < next and Nat.0 < third implies
        Nat.0 < continued_fraction_convergent(
            List.cons(head, List.cons(next, List.cons(third, List.nil[Nat])))).second
} by {
    if Nat.0 < next and Nat.0 < third {
        continued_fraction_convergent_triple_second_eq_denominator(head, next, third)
        continued_fraction_denominator_triple_positive_of_tail_positive(
            head, next, third)
        Nat.0 < continued_fraction_denominator(
            List.cons(head, List.cons(next, List.cons(third, List.nil[Nat]))))
        Nat.0 < continued_fraction_convergent(
            List.cons(head, List.cons(next, List.cons(third, List.nil[Nat])))).second
    }
}

/// A valid three-term convergent has positive denominator projection.
theorem continued_fraction_convergent_triple_second_positive(
    head: Nat, next: Nat, third: Nat
) {
    finite_continued_fraction_coefficients(
        List.cons(head, List.cons(next, List.cons(third, List.nil[Nat]))))
        implies Nat.0 < continued_fraction_convergent(
            List.cons(head, List.cons(next, List.cons(third, List.nil[Nat])))).second
} by {
    if finite_continued_fraction_coefficients(
        List.cons(head, List.cons(next, List.cons(third, List.nil[Nat])))) {
        finite_continued_fraction_coefficients_triple_second_positive(head, next, third)
        finite_continued_fraction_coefficients_triple_third_positive(head, next, third)
        continued_fraction_convergent_triple_second_positive_of_tail_positive(
            head, next, third)
    }
}

/// A valid three-term convergent has nonzero denominator projection.
theorem continued_fraction_convergent_triple_second_ne_zero(
    head: Nat, next: Nat, third: Nat
) {
    finite_continued_fraction_coefficients(
        List.cons(head, List.cons(next, List.cons(third, List.nil[Nat]))))
        implies continued_fraction_convergent(
            List.cons(head, List.cons(next, List.cons(third, List.nil[Nat])))).second != Nat.0
} by {
    if finite_continued_fraction_coefficients(
        List.cons(head, List.cons(next, List.cons(third, List.nil[Nat])))) {
        continued_fraction_convergent_triple_second_positive(head, next, third)
        nat_positive_ne_zero(continued_fraction_convergent(
            List.cons(head, List.cons(next, List.cons(third, List.nil[Nat])))).second)
    }
}

/// A singleton continued fraction has convergent pair `head/1`.
theorem continued_fraction_singleton_convergent_eq_pair(head: Nat) {
    ContinuedFraction.singleton(head).convergent = Pair.new(head, Nat.1)
} by {
    continued_fraction_singleton_coefficients(head)
    continued_fraction_convergent_eq_coefficients_convergent(ContinuedFraction.singleton(head))
    continued_fraction_convergent_singleton_eq_pair(head)
}

/// A singleton continued fraction has convergent first projection equal to its
/// numerator method.
theorem continued_fraction_singleton_convergent_first_eq_numerator(head: Nat) {
    ContinuedFraction.singleton(head).convergent.first =
        ContinuedFraction.singleton(head).numerator
} by {
    continued_fraction_singleton_convergent_first(head)
    continued_fraction_singleton_numerator(head)
}

/// A singleton continued fraction has convergent second projection equal to
/// its denominator method.
theorem continued_fraction_singleton_convergent_second_eq_denominator(head: Nat) {
    ContinuedFraction.singleton(head).convergent.second =
        ContinuedFraction.singleton(head).denominator
} by {
    continued_fraction_singleton_convergent_second(head)
    continued_fraction_singleton_denominator(head)
}

/// A two-term continued fraction has numerator equal to the two-term
/// continuant.
theorem continued_fraction_pair_numerator(cf: ContinuedFraction,
    head: Nat, next: Nat) {
    cf.coefficients = List.cons(head, List.cons(next, List.nil[Nat]))
        implies cf.numerator = head * next + Nat.1
} by {
    if cf.coefficients = List.cons(head, List.cons(next, List.nil[Nat])) {
        continued_fraction_numerator_eq_coefficients_numerator(cf)
        continued_fraction_numerator_pair(head, next)
    }
}

/// A two-term continued fraction has convergent first projection equal to the
/// two-term numerator.
theorem continued_fraction_pair_convergent_first(cf: ContinuedFraction,
    head: Nat, next: Nat) {
    cf.coefficients = List.cons(head, List.cons(next, List.nil[Nat]))
        implies cf.convergent.first = head * next + Nat.1
} by {
    if cf.coefficients = List.cons(head, List.cons(next, List.nil[Nat])) {
        continued_fraction_convergent_eq_coefficients_convergent(cf)
        continued_fraction_convergent_pair_first(head, next)
    }
}

/// A two-term continued fraction has convergent second projection equal to the
/// second coefficient.
theorem continued_fraction_pair_convergent_second(cf: ContinuedFraction,
    head: Nat, next: Nat) {
    cf.coefficients = List.cons(head, List.cons(next, List.nil[Nat]))
        implies cf.convergent.second = next
} by {
    if cf.coefficients = List.cons(head, List.cons(next, List.nil[Nat])) {
        continued_fraction_convergent_eq_coefficients_convergent(cf)
        continued_fraction_convergent_pair_second(head, next)
    }
}

/// A two-term continued fraction has convergent first projection equal to its
/// numerator method.
theorem continued_fraction_pair_convergent_first_eq_numerator(
    cf: ContinuedFraction, head: Nat, next: Nat
) {
    cf.coefficients = List.cons(head, List.cons(next, List.nil[Nat]))
        implies cf.convergent.first = cf.numerator
} by {
    if cf.coefficients = List.cons(head, List.cons(next, List.nil[Nat])) {
        continued_fraction_pair_convergent_first(cf, head, next)
        continued_fraction_pair_numerator(cf, head, next)
    }
}

/// A two-term continued fraction has convergent second projection equal to its
/// denominator method.
theorem continued_fraction_pair_convergent_second_eq_denominator(
    cf: ContinuedFraction, head: Nat, next: Nat
) {
    cf.coefficients = List.cons(head, List.cons(next, List.nil[Nat]))
        implies cf.convergent.second = cf.denominator
} by {
    if cf.coefficients = List.cons(head, List.cons(next, List.nil[Nat])) {
        continued_fraction_pair_convergent_second(cf, head, next)
        continued_fraction_pair_denominator(cf, head, next)
    }
}

/// A two-term continued fraction has convergent pair formed by its explicit
/// numerator and denominator.
theorem continued_fraction_pair_convergent_eq_pair(cf: ContinuedFraction,
    head: Nat, next: Nat) {
    cf.coefficients = List.cons(head, List.cons(next, List.nil[Nat]))
        implies cf.convergent = Pair.new(head * next + Nat.1, next)
} by {
    if cf.coefficients = List.cons(head, List.cons(next, List.nil[Nat])) {
        continued_fraction_convergent_eq_coefficients_convergent(cf)
        continued_fraction_convergent_pair_eq_pair(head, next)
    }
}

/// A two-term continued fraction has positive convergent denominator.
theorem continued_fraction_pair_convergent_second_positive(cf: ContinuedFraction,
    head: Nat, next: Nat) {
    cf.coefficients = List.cons(head, List.cons(next, List.nil[Nat]))
        implies Nat.0 < cf.convergent.second
} by {
    if cf.coefficients = List.cons(head, List.cons(next, List.nil[Nat])) {
        continued_fraction_coefficients_valid(cf)
        finite_continued_fraction_coefficients(cf.coefficients)
        finite_continued_fraction_coefficients(List.cons(head, List.cons(next, List.nil[Nat])))
        continued_fraction_convergent_pair_second_positive(head, next)
        continued_fraction_convergent_eq_coefficients_convergent(cf)
        Nat.0 < continued_fraction_convergent(
            List.cons(head, List.cons(next, List.nil[Nat]))).second
        Nat.0 < cf.convergent.second
    }
}

/// A two-term continued fraction has nonzero convergent denominator.
theorem continued_fraction_pair_convergent_second_ne_zero(cf: ContinuedFraction,
    head: Nat, next: Nat) {
    cf.coefficients = List.cons(head, List.cons(next, List.nil[Nat]))
        implies cf.convergent.second != Nat.0
} by {
    if cf.coefficients = List.cons(head, List.cons(next, List.nil[Nat])) {
        continued_fraction_pair_convergent_second_positive(cf, head, next)
        nat_positive_ne_zero(cf.convergent.second)
    }
}

/// A three-term continued fraction has numerator equal to the three-term
/// continuant.
theorem continued_fraction_triple_numerator(cf: ContinuedFraction,
    head: Nat, next: Nat, third: Nat) {
    cf.coefficients = List.cons(head, List.cons(next, List.cons(third, List.nil[Nat])))
        implies cf.numerator = (head * next + Nat.1) * third + head
} by {
    if cf.coefficients = List.cons(head, List.cons(next, List.cons(third, List.nil[Nat]))) {
        continued_fraction_numerator_eq_coefficients_numerator(cf)
        continued_fraction_numerator_triple(head, next, third)
    }
}

/// A three-term continued fraction has convergent first projection equal to
/// the three-term numerator.
theorem continued_fraction_triple_convergent_first(cf: ContinuedFraction,
    head: Nat, next: Nat, third: Nat) {
    cf.coefficients = List.cons(head, List.cons(next, List.cons(third, List.nil[Nat])))
        implies cf.convergent.first = (head * next + Nat.1) * third + head
} by {
    if cf.coefficients = List.cons(head, List.cons(next, List.cons(third, List.nil[Nat]))) {
        continued_fraction_convergent_eq_coefficients_convergent(cf)
        continued_fraction_convergent_triple_first(head, next, third)
    }
}

/// A three-term continued fraction has convergent second projection equal to
/// the tail continuant.
theorem continued_fraction_triple_convergent_second(cf: ContinuedFraction,
    head: Nat, next: Nat, third: Nat) {
    cf.coefficients = List.cons(head, List.cons(next, List.cons(third, List.nil[Nat])))
        implies cf.convergent.second = next * third + Nat.1
} by {
    if cf.coefficients = List.cons(head, List.cons(next, List.cons(third, List.nil[Nat]))) {
        continued_fraction_convergent_eq_coefficients_convergent(cf)
        continued_fraction_convergent_triple_second(head, next, third)
    }
}

/// A three-term continued fraction has convergent first projection equal to
/// its numerator method.
theorem continued_fraction_triple_convergent_first_eq_numerator(
    cf: ContinuedFraction, head: Nat, next: Nat, third: Nat
) {
    cf.coefficients = List.cons(head, List.cons(next, List.cons(third, List.nil[Nat])))
        implies cf.convergent.first = cf.numerator
} by {
    if cf.coefficients = List.cons(head, List.cons(next, List.cons(third, List.nil[Nat]))) {
        continued_fraction_triple_convergent_first(cf, head, next, third)
        continued_fraction_triple_numerator(cf, head, next, third)
    }
}

/// A three-term continued fraction has convergent second projection equal to
/// its denominator method.
theorem continued_fraction_triple_convergent_second_eq_denominator(
    cf: ContinuedFraction, head: Nat, next: Nat, third: Nat
) {
    cf.coefficients = List.cons(head, List.cons(next, List.cons(third, List.nil[Nat])))
        implies cf.convergent.second = cf.denominator
} by {
    if cf.coefficients = List.cons(head, List.cons(next, List.cons(third, List.nil[Nat]))) {
        continued_fraction_triple_convergent_second(cf, head, next, third)
        continued_fraction_triple_denominator(cf, head, next, third)
    }
}

/// A three-term continued fraction has convergent pair formed by its explicit
/// numerator and denominator.
theorem continued_fraction_triple_convergent_eq_pair(cf: ContinuedFraction,
    head: Nat, next: Nat, third: Nat) {
    cf.coefficients = List.cons(head, List.cons(next, List.cons(third, List.nil[Nat])))
        implies cf.convergent =
            Pair.new((head * next + Nat.1) * third + head, next * third + Nat.1)
} by {
    if cf.coefficients = List.cons(head, List.cons(next, List.cons(third, List.nil[Nat]))) {
        continued_fraction_convergent_eq_coefficients_convergent(cf)
        continued_fraction_convergent_triple_eq_pair(head, next, third)
    }
}

/// A three-term continued fraction has positive convergent denominator.
theorem continued_fraction_triple_convergent_second_positive(cf: ContinuedFraction,
    head: Nat, next: Nat, third: Nat) {
    cf.coefficients = List.cons(head, List.cons(next, List.cons(third, List.nil[Nat])))
        implies Nat.0 < cf.convergent.second
} by {
    if cf.coefficients = List.cons(head, List.cons(next, List.cons(third, List.nil[Nat]))) {
        continued_fraction_coefficients_valid(cf)
        finite_continued_fraction_coefficients(cf.coefficients)
        finite_continued_fraction_coefficients(
            List.cons(head, List.cons(next, List.cons(third, List.nil[Nat]))))
        continued_fraction_convergent_triple_second_positive(head, next, third)
        continued_fraction_convergent_eq_coefficients_convergent(cf)
        Nat.0 < continued_fraction_convergent(
            List.cons(head, List.cons(next, List.cons(third, List.nil[Nat])))).second
        Nat.0 < cf.convergent.second
    }
}

/// A three-term continued fraction has nonzero convergent denominator.
theorem continued_fraction_triple_convergent_second_ne_zero(cf: ContinuedFraction,
    head: Nat, next: Nat, third: Nat) {
    cf.coefficients = List.cons(head, List.cons(next, List.cons(third, List.nil[Nat])))
        implies cf.convergent.second != Nat.0
} by {
    if cf.coefficients = List.cons(head, List.cons(next, List.cons(third, List.nil[Nat]))) {
        continued_fraction_triple_convergent_second_positive(cf, head, next, third)
        nat_positive_ne_zero(cf.convergent.second)
    }
}

/// Adding a positive natural to the right gives a positive natural.
theorem nat_add_positive_right(left: Nat, right: Nat) {
    Nat.0 < right implies Nat.0 < left + right
} by {
    if Nat.0 < right {
        nat_add_positive_left(right, left)
        Nat.0 < right + left
        right + left = left + right
        Nat.0 < left + right
    }
}

/// Adding one to a natural number gives a positive natural.
theorem nat_add_one_positive(n: Nat) {
    Nat.0 < n + Nat.1
} by {
    Nat.0 < Nat.1
    nat_add_positive_right(n, Nat.1)
}

/// The empty numerator is the first projection of the empty convergent.
theorem continued_fraction_numerator_nil_eq_convergent_first {
    continued_fraction_numerator(List.nil[Nat]) =
        continued_fraction_convergent(List.nil[Nat]).first
} by {
    continued_fraction_convergent_nil_first_eq_numerator
}

/// The empty denominator is the second projection of the empty convergent.
theorem continued_fraction_denominator_nil_eq_convergent_second {
    continued_fraction_denominator(List.nil[Nat]) =
        continued_fraction_convergent(List.nil[Nat]).second
} by {
    continued_fraction_convergent_nil_second_eq_denominator
}

/// The empty convergent is the pair formed by the empty numerator and
/// denominator.
theorem continued_fraction_convergent_nil_eq_numerator_denominator_pair {
    continued_fraction_convergent(List.nil[Nat]) =
        Pair.new(continued_fraction_numerator(List.nil[Nat]),
            continued_fraction_denominator(List.nil[Nat]))
} by {
    continued_fraction_convergent_nil_first_eq_numerator
    continued_fraction_convergent_nil_second_eq_denominator
    pair_new_first(continued_fraction_numerator(List.nil[Nat]),
        continued_fraction_denominator(List.nil[Nat]))
    pair_new_second(continued_fraction_numerator(List.nil[Nat]),
        continued_fraction_denominator(List.nil[Nat]))
    pair_ext(continued_fraction_convergent(List.nil[Nat]),
        Pair.new(continued_fraction_numerator(List.nil[Nat]),
            continued_fraction_denominator(List.nil[Nat])))
}

/// The singleton numerator is the first projection of the singleton
/// convergent.
theorem continued_fraction_numerator_singleton_eq_convergent_first(head: Nat) {
    continued_fraction_numerator(List.cons(head, List.nil[Nat])) =
        continued_fraction_convergent(List.cons(head, List.nil[Nat])).first
} by {
    continued_fraction_convergent_singleton_first_eq_numerator(head)
}

/// The singleton denominator is the second projection of the singleton
/// convergent.
theorem continued_fraction_denominator_singleton_eq_convergent_second(head: Nat) {
    continued_fraction_denominator(List.cons(head, List.nil[Nat])) =
        continued_fraction_convergent(List.cons(head, List.nil[Nat])).second
} by {
    continued_fraction_convergent_singleton_second_eq_denominator(head)
}

/// The singleton convergent is the pair formed by its numerator and
/// denominator definitions.
theorem continued_fraction_convergent_singleton_eq_numerator_denominator_pair(head: Nat) {
    continued_fraction_convergent(List.cons(head, List.nil[Nat])) =
        Pair.new(continued_fraction_numerator(List.cons(head, List.nil[Nat])),
            continued_fraction_denominator(List.cons(head, List.nil[Nat])))
} by {
    continued_fraction_convergent_singleton_first_eq_numerator(head)
    continued_fraction_convergent_singleton_second_eq_denominator(head)
    pair_new_first(continued_fraction_numerator(List.cons(head, List.nil[Nat])),
        continued_fraction_denominator(List.cons(head, List.nil[Nat])))
    pair_new_second(continued_fraction_numerator(List.cons(head, List.nil[Nat])),
        continued_fraction_denominator(List.cons(head, List.nil[Nat])))
    pair_ext(continued_fraction_convergent(List.cons(head, List.nil[Nat])),
        Pair.new(continued_fraction_numerator(List.cons(head, List.nil[Nat])),
            continued_fraction_denominator(List.cons(head, List.nil[Nat]))))
}

/// The two-term numerator is the first projection of the two-term convergent.
theorem continued_fraction_numerator_pair_eq_convergent_first(head: Nat, next: Nat) {
    continued_fraction_numerator(List.cons(head, List.cons(next, List.nil[Nat]))) =
        continued_fraction_convergent(List.cons(head, List.cons(next, List.nil[Nat]))).first
} by {
    continued_fraction_convergent_pair_first_eq_numerator(head, next)
}

/// The two-term denominator is the second projection of the two-term
/// convergent.
theorem continued_fraction_denominator_pair_eq_convergent_second(head: Nat, next: Nat) {
    continued_fraction_denominator(List.cons(head, List.cons(next, List.nil[Nat]))) =
        continued_fraction_convergent(List.cons(head, List.cons(next, List.nil[Nat]))).second
} by {
    continued_fraction_convergent_pair_second_eq_denominator(head, next)
}

/// The two-term convergent is the pair formed by its numerator and denominator
/// definitions.
theorem continued_fraction_convergent_pair_eq_numerator_denominator_pair(
    head: Nat, next: Nat
) {
    continued_fraction_convergent(List.cons(head, List.cons(next, List.nil[Nat]))) =
        Pair.new(continued_fraction_numerator(List.cons(head, List.cons(next, List.nil[Nat]))),
            continued_fraction_denominator(List.cons(head, List.cons(next, List.nil[Nat]))))
} by {
    continued_fraction_convergent_pair_first_eq_numerator(head, next)
    continued_fraction_convergent_pair_second_eq_denominator(head, next)
    pair_new_first(continued_fraction_numerator(
        List.cons(head, List.cons(next, List.nil[Nat]))),
        continued_fraction_denominator(List.cons(head, List.cons(next, List.nil[Nat]))))
    pair_new_second(continued_fraction_numerator(
        List.cons(head, List.cons(next, List.nil[Nat]))),
        continued_fraction_denominator(List.cons(head, List.cons(next, List.nil[Nat]))))
    pair_ext(continued_fraction_convergent(
        List.cons(head, List.cons(next, List.nil[Nat]))),
        Pair.new(continued_fraction_numerator(
            List.cons(head, List.cons(next, List.nil[Nat]))),
            continued_fraction_denominator(List.cons(head, List.cons(next, List.nil[Nat])))))
}

/// The three-term numerator is the first projection of the three-term
/// convergent.
theorem continued_fraction_numerator_triple_eq_convergent_first(
    head: Nat, next: Nat, third: Nat
) {
    continued_fraction_numerator(
        List.cons(head, List.cons(next, List.cons(third, List.nil[Nat])))) =
        continued_fraction_convergent(
            List.cons(head, List.cons(next, List.cons(third, List.nil[Nat])))).first
} by {
    continued_fraction_convergent_triple_first_eq_numerator(head, next, third)
}

/// The three-term denominator is the second projection of the three-term
/// convergent.
theorem continued_fraction_denominator_triple_eq_convergent_second(
    head: Nat, next: Nat, third: Nat
) {
    continued_fraction_denominator(
        List.cons(head, List.cons(next, List.cons(third, List.nil[Nat])))) =
        continued_fraction_convergent(
            List.cons(head, List.cons(next, List.cons(third, List.nil[Nat])))).second
} by {
    continued_fraction_convergent_triple_second_eq_denominator(head, next, third)
}

/// The three-term convergent is the pair formed by its numerator and
/// denominator definitions.
theorem continued_fraction_convergent_triple_eq_numerator_denominator_pair(
    head: Nat, next: Nat, third: Nat
) {
    continued_fraction_convergent(
        List.cons(head, List.cons(next, List.cons(third, List.nil[Nat])))) =
        Pair.new(continued_fraction_numerator(
                List.cons(head, List.cons(next, List.cons(third, List.nil[Nat])))),
            continued_fraction_denominator(
                List.cons(head, List.cons(next, List.cons(third, List.nil[Nat])))))
} by {
    continued_fraction_convergent_triple_first_eq_numerator(head, next, third)
    continued_fraction_convergent_triple_second_eq_denominator(head, next, third)
    pair_new_first(continued_fraction_numerator(
        List.cons(head, List.cons(next, List.cons(third, List.nil[Nat])))),
        continued_fraction_denominator(
            List.cons(head, List.cons(next, List.cons(third, List.nil[Nat])))))
    pair_new_second(continued_fraction_numerator(
        List.cons(head, List.cons(next, List.cons(third, List.nil[Nat])))),
        continued_fraction_denominator(
            List.cons(head, List.cons(next, List.cons(third, List.nil[Nat])))))
    pair_ext(continued_fraction_convergent(
        List.cons(head, List.cons(next, List.cons(third, List.nil[Nat])))),
        Pair.new(continued_fraction_numerator(
                List.cons(head, List.cons(next, List.cons(third, List.nil[Nat])))),
            continued_fraction_denominator(
                List.cons(head, List.cons(next, List.cons(third, List.nil[Nat]))))))
}

/// A singleton continued fraction has convergent pair formed by its numerator
/// and denominator methods.
theorem continued_fraction_singleton_convergent_eq_numerator_denominator_pair(
    head: Nat
) {
    ContinuedFraction.singleton(head).convergent =
        Pair.new(ContinuedFraction.singleton(head).numerator,
            ContinuedFraction.singleton(head).denominator)
} by {
    continued_fraction_singleton_convergent_first_eq_numerator(head)
    continued_fraction_singleton_convergent_second_eq_denominator(head)
    pair_new_first(ContinuedFraction.singleton(head).numerator,
        ContinuedFraction.singleton(head).denominator)
    pair_new_second(ContinuedFraction.singleton(head).numerator,
        ContinuedFraction.singleton(head).denominator)
    pair_ext(ContinuedFraction.singleton(head).convergent,
        Pair.new(ContinuedFraction.singleton(head).numerator,
            ContinuedFraction.singleton(head).denominator))
}

/// A two-term continued fraction has numerator method equal to the convergent
/// first projection.
theorem continued_fraction_pair_numerator_eq_convergent_first(
    cf: ContinuedFraction, head: Nat, next: Nat
) {
    cf.coefficients = List.cons(head, List.cons(next, List.nil[Nat]))
        implies cf.numerator = cf.convergent.first
} by {
    if cf.coefficients = List.cons(head, List.cons(next, List.nil[Nat])) {
        continued_fraction_pair_convergent_first_eq_numerator(cf, head, next)
    }
}

/// A two-term continued fraction has denominator method equal to the
/// convergent second projection.
theorem continued_fraction_pair_denominator_eq_convergent_second(
    cf: ContinuedFraction, head: Nat, next: Nat
) {
    cf.coefficients = List.cons(head, List.cons(next, List.nil[Nat]))
        implies cf.denominator = cf.convergent.second
} by {
    if cf.coefficients = List.cons(head, List.cons(next, List.nil[Nat])) {
        continued_fraction_pair_convergent_second_eq_denominator(cf, head, next)
    }
}

/// A three-term continued fraction has numerator method equal to the
/// convergent first projection.
theorem continued_fraction_triple_numerator_eq_convergent_first(
    cf: ContinuedFraction, head: Nat, next: Nat, third: Nat
) {
    cf.coefficients = List.cons(head, List.cons(next, List.cons(third, List.nil[Nat])))
        implies cf.numerator = cf.convergent.first
} by {
    if cf.coefficients = List.cons(head, List.cons(next, List.cons(third, List.nil[Nat]))) {
        continued_fraction_triple_convergent_first_eq_numerator(cf, head, next, third)
    }
}

/// A three-term continued fraction has denominator method equal to the
/// convergent second projection.
theorem continued_fraction_triple_denominator_eq_convergent_second(
    cf: ContinuedFraction, head: Nat, next: Nat, third: Nat
) {
    cf.coefficients = List.cons(head, List.cons(next, List.cons(third, List.nil[Nat])))
        implies cf.denominator = cf.convergent.second
} by {
    if cf.coefficients = List.cons(head, List.cons(next, List.cons(third, List.nil[Nat]))) {
        continued_fraction_triple_convergent_second_eq_denominator(cf, head, next, third)
    }
}

/// The two-term numerator is positive.
theorem continued_fraction_numerator_pair_positive(head: Nat, next: Nat) {
    Nat.0 < continued_fraction_numerator(
        List.cons(head, List.cons(next, List.nil[Nat])))
} by {
    continued_fraction_numerator_pair(head, next)
    nat_add_one_positive(head * next)
}

/// The two-term numerator is nonzero.
theorem continued_fraction_numerator_pair_ne_zero(head: Nat, next: Nat) {
    continued_fraction_numerator(
        List.cons(head, List.cons(next, List.nil[Nat]))) != Nat.0
} by {
    continued_fraction_numerator_pair_positive(head, next)
    nat_positive_ne_zero(continued_fraction_numerator(
        List.cons(head, List.cons(next, List.nil[Nat]))))
}

/// The two-term convergent numerator projection is positive.
theorem continued_fraction_convergent_pair_first_positive(head: Nat, next: Nat) {
    Nat.0 < continued_fraction_convergent(
        List.cons(head, List.cons(next, List.nil[Nat]))).first
} by {
    continued_fraction_convergent_pair_first_eq_numerator(head, next)
    continued_fraction_numerator_pair_positive(head, next)
}

/// The two-term convergent numerator projection is nonzero.
theorem continued_fraction_convergent_pair_first_ne_zero(head: Nat, next: Nat) {
    continued_fraction_convergent(
        List.cons(head, List.cons(next, List.nil[Nat]))).first != Nat.0
} by {
    continued_fraction_convergent_pair_first_positive(head, next)
    nat_positive_ne_zero(continued_fraction_convergent(
        List.cons(head, List.cons(next, List.nil[Nat]))).first)
}

/// A two-term continued fraction has positive numerator method.
theorem continued_fraction_pair_numerator_positive(cf: ContinuedFraction,
    head: Nat, next: Nat) {
    cf.coefficients = List.cons(head, List.cons(next, List.nil[Nat]))
        implies Nat.0 < cf.numerator
} by {
    if cf.coefficients = List.cons(head, List.cons(next, List.nil[Nat])) {
        continued_fraction_pair_numerator(cf, head, next)
        continued_fraction_numerator_pair_positive(head, next)
    }
}

/// A two-term continued fraction has nonzero numerator method.
theorem continued_fraction_pair_numerator_ne_zero(cf: ContinuedFraction,
    head: Nat, next: Nat) {
    cf.coefficients = List.cons(head, List.cons(next, List.nil[Nat]))
        implies cf.numerator != Nat.0
} by {
    if cf.coefficients = List.cons(head, List.cons(next, List.nil[Nat])) {
        continued_fraction_pair_numerator_positive(cf, head, next)
        nat_positive_ne_zero(cf.numerator)
    }
}

/// A three-term numerator is positive when the third coefficient is positive.
theorem continued_fraction_numerator_triple_positive_of_third_positive(
    head: Nat, next: Nat, third: Nat
) {
    Nat.0 < third implies Nat.0 < continued_fraction_numerator(
        List.cons(head, List.cons(next, List.cons(third, List.nil[Nat]))))
} by {
    if Nat.0 < third {
        continued_fraction_numerator_triple(head, next, third)
        nat_add_one_positive(head * next)
        Nat.0 < head * next + Nat.1
        nat_mul_positive(head * next + Nat.1, third)
        Nat.0 < (head * next + Nat.1) * third
        nat_add_positive_left((head * next + Nat.1) * third, head)
        Nat.0 < (head * next + Nat.1) * third + head
        Nat.0 < continued_fraction_numerator(
            List.cons(head, List.cons(next, List.cons(third, List.nil[Nat]))))
    }
}

/// A valid three-term numerator is positive.
theorem continued_fraction_numerator_triple_positive(
    head: Nat, next: Nat, third: Nat
) {
    finite_continued_fraction_coefficients(
        List.cons(head, List.cons(next, List.cons(third, List.nil[Nat]))))
        implies Nat.0 < continued_fraction_numerator(
            List.cons(head, List.cons(next, List.cons(third, List.nil[Nat]))))
} by {
    if finite_continued_fraction_coefficients(
        List.cons(head, List.cons(next, List.cons(third, List.nil[Nat])))) {
        finite_continued_fraction_coefficients_triple_third_positive(head, next, third)
        continued_fraction_numerator_triple_positive_of_third_positive(head, next, third)
    }
}

/// A valid three-term numerator is nonzero.
theorem continued_fraction_numerator_triple_ne_zero(
    head: Nat, next: Nat, third: Nat
) {
    finite_continued_fraction_coefficients(
        List.cons(head, List.cons(next, List.cons(third, List.nil[Nat]))))
        implies continued_fraction_numerator(
            List.cons(head, List.cons(next, List.cons(third, List.nil[Nat])))) != Nat.0
} by {
    if finite_continued_fraction_coefficients(
        List.cons(head, List.cons(next, List.cons(third, List.nil[Nat])))) {
        continued_fraction_numerator_triple_positive(head, next, third)
        nat_positive_ne_zero(continued_fraction_numerator(
            List.cons(head, List.cons(next, List.cons(third, List.nil[Nat])))))
    }
}

/// A three-term continued fraction has positive numerator method.
theorem continued_fraction_triple_numerator_positive(cf: ContinuedFraction,
    head: Nat, next: Nat, third: Nat) {
    cf.coefficients = List.cons(head, List.cons(next, List.cons(third, List.nil[Nat])))
        implies Nat.0 < cf.numerator
} by {
    if cf.coefficients = List.cons(head, List.cons(next, List.cons(third, List.nil[Nat]))) {
        continued_fraction_coefficients_valid(cf)
        finite_continued_fraction_coefficients(cf.coefficients)
        finite_continued_fraction_coefficients(
            List.cons(head, List.cons(next, List.cons(third, List.nil[Nat]))))
        continued_fraction_numerator_triple_positive(head, next, third)
        continued_fraction_triple_numerator(cf, head, next, third)
        Nat.0 < cf.numerator
    }
}

/// A three-term continued fraction has nonzero numerator method.
theorem continued_fraction_triple_numerator_ne_zero(cf: ContinuedFraction,
    head: Nat, next: Nat, third: Nat) {
    cf.coefficients = List.cons(head, List.cons(next, List.cons(third, List.nil[Nat])))
        implies cf.numerator != Nat.0
} by {
    if cf.coefficients = List.cons(head, List.cons(next, List.cons(third, List.nil[Nat]))) {
        continued_fraction_triple_numerator_positive(cf, head, next, third)
        nat_positive_ne_zero(cf.numerator)
    }
}

/// The singleton numerator is the singleton continuant.
theorem continued_fraction_numerator_singleton_eq_continuant(head: Nat) {
    continued_fraction_numerator(List.cons(head, List.nil[Nat])) =
        continuant(List.cons(head, List.nil[Nat]))
} by {
    continued_fraction_numerator_singleton(head)
    continuant_singleton(head)
}

/// The two-term numerator is the two-term continuant.
theorem continued_fraction_numerator_pair_eq_continuant(head: Nat, next: Nat) {
    continued_fraction_numerator(List.cons(head, List.cons(next, List.nil[Nat]))) =
        continuant(List.cons(head, List.cons(next, List.nil[Nat])))
} by {
    continued_fraction_numerator_pair(head, next)
    continuant_pair(head, next)
}

/// The three-term numerator is the three-term continuant.
theorem continued_fraction_numerator_triple_eq_continuant(
    head: Nat, next: Nat, third: Nat
) {
    continued_fraction_numerator(
        List.cons(head, List.cons(next, List.cons(third, List.nil[Nat])))) =
        continuant(List.cons(head, List.cons(next, List.cons(third, List.nil[Nat]))))
} by {
    continued_fraction_numerator_triple(head, next, third)
    continuant_triple(head, next, third)
}

/// A singleton continued fraction has numerator equal to its continuant
/// method.
theorem continued_fraction_singleton_numerator_eq_continuant(head: Nat) {
    ContinuedFraction.singleton(head).numerator =
        ContinuedFraction.singleton(head).continuant
} by {
    continued_fraction_singleton_numerator(head)
    continued_fraction_singleton_continuant(head)
}

/// A two-term continued fraction has continuant equal to the two-term
/// numerator.
theorem continued_fraction_pair_continuant(cf: ContinuedFraction,
    head: Nat, next: Nat) {
    cf.coefficients = List.cons(head, List.cons(next, List.nil[Nat]))
        implies cf.continuant = head * next + Nat.1
} by {
    if cf.coefficients = List.cons(head, List.cons(next, List.nil[Nat])) {
        continued_fraction_continuant_eq_coefficients_continuant(cf)
        continuant_pair(head, next)
    }
}

/// A two-term continued fraction has numerator equal to its continuant method.
theorem continued_fraction_pair_numerator_eq_continuant(cf: ContinuedFraction,
    head: Nat, next: Nat) {
    cf.coefficients = List.cons(head, List.cons(next, List.nil[Nat]))
        implies cf.numerator = cf.continuant
} by {
    if cf.coefficients = List.cons(head, List.cons(next, List.nil[Nat])) {
        continued_fraction_pair_numerator(cf, head, next)
        continued_fraction_pair_continuant(cf, head, next)
    }
}

/// A three-term continued fraction has continuant equal to the three-term
/// numerator.
theorem continued_fraction_triple_continuant(cf: ContinuedFraction,
    head: Nat, next: Nat, third: Nat) {
    cf.coefficients = List.cons(head, List.cons(next, List.cons(third, List.nil[Nat])))
        implies cf.continuant = (head * next + Nat.1) * third + head
} by {
    if cf.coefficients = List.cons(head, List.cons(next, List.cons(third, List.nil[Nat]))) {
        continued_fraction_continuant_eq_coefficients_continuant(cf)
        continuant_triple(head, next, third)
    }
}

/// A three-term continued fraction has numerator equal to its continuant
/// method.
theorem continued_fraction_triple_numerator_eq_continuant(cf: ContinuedFraction,
    head: Nat, next: Nat, third: Nat) {
    cf.coefficients = List.cons(head, List.cons(next, List.cons(third, List.nil[Nat])))
        implies cf.numerator = cf.continuant
} by {
    if cf.coefficients = List.cons(head, List.cons(next, List.cons(third, List.nil[Nat]))) {
        continued_fraction_triple_numerator(cf, head, next, third)
        continued_fraction_triple_continuant(cf, head, next, third)
    }
}

/// The two-term continuant is positive.
theorem continuant_pair_positive(head: Nat, next: Nat) {
    Nat.0 < continuant(List.cons(head, List.cons(next, List.nil[Nat])))
} by {
    continuant_pair(head, next)
    nat_add_one_positive(head * next)
}

/// The two-term continuant is nonzero.
theorem continuant_pair_ne_zero(head: Nat, next: Nat) {
    continuant(List.cons(head, List.cons(next, List.nil[Nat]))) != Nat.0
} by {
    continuant_pair_positive(head, next)
    nat_positive_ne_zero(continuant(List.cons(head, List.cons(next, List.nil[Nat]))))
}

/// A two-term continued fraction has positive continuant method.
theorem continued_fraction_pair_continuant_positive(cf: ContinuedFraction,
    head: Nat, next: Nat) {
    cf.coefficients = List.cons(head, List.cons(next, List.nil[Nat]))
        implies Nat.0 < cf.continuant
} by {
    if cf.coefficients = List.cons(head, List.cons(next, List.nil[Nat])) {
        continued_fraction_pair_continuant(cf, head, next)
        continuant_pair_positive(head, next)
    }
}

/// A two-term continued fraction has nonzero continuant method.
theorem continued_fraction_pair_continuant_ne_zero(cf: ContinuedFraction,
    head: Nat, next: Nat) {
    cf.coefficients = List.cons(head, List.cons(next, List.nil[Nat]))
        implies cf.continuant != Nat.0
} by {
    if cf.coefficients = List.cons(head, List.cons(next, List.nil[Nat])) {
        continued_fraction_pair_continuant_positive(cf, head, next)
        nat_positive_ne_zero(cf.continuant)
    }
}

/// The three-term continuant is positive when the third coefficient is
/// positive.
theorem continuant_triple_positive_of_third_positive(
    head: Nat, next: Nat, third: Nat
) {
    Nat.0 < third implies
        Nat.0 < continuant(List.cons(head, List.cons(next, List.cons(third, List.nil[Nat]))))
} by {
    if Nat.0 < third {
        continued_fraction_numerator_triple_positive_of_third_positive(
            head, next, third)
        continued_fraction_numerator_triple_eq_continuant(head, next, third)
        Nat.0 < continuant(
            List.cons(head, List.cons(next, List.cons(third, List.nil[Nat]))))
    }
}

/// A valid three-term continuant is positive.
theorem continuant_triple_positive(head: Nat, next: Nat, third: Nat) {
    finite_continued_fraction_coefficients(
        List.cons(head, List.cons(next, List.cons(third, List.nil[Nat]))))
        implies Nat.0 < continuant(
            List.cons(head, List.cons(next, List.cons(third, List.nil[Nat]))))
} by {
    if finite_continued_fraction_coefficients(
        List.cons(head, List.cons(next, List.cons(third, List.nil[Nat])))) {
        finite_continued_fraction_coefficients_triple_third_positive(head, next, third)
        continuant_triple_positive_of_third_positive(head, next, third)
    }
}

/// A valid three-term continuant is nonzero.
theorem continuant_triple_ne_zero(head: Nat, next: Nat, third: Nat) {
    finite_continued_fraction_coefficients(
        List.cons(head, List.cons(next, List.cons(third, List.nil[Nat]))))
        implies continuant(
            List.cons(head, List.cons(next, List.cons(third, List.nil[Nat])))) != Nat.0
} by {
    if finite_continued_fraction_coefficients(
        List.cons(head, List.cons(next, List.cons(third, List.nil[Nat])))) {
        continuant_triple_positive(head, next, third)
        nat_positive_ne_zero(continuant(
            List.cons(head, List.cons(next, List.cons(third, List.nil[Nat])))))
    }
}

/// A three-term continued fraction has positive continuant method.
theorem continued_fraction_triple_continuant_positive(cf: ContinuedFraction,
    head: Nat, next: Nat, third: Nat) {
    cf.coefficients = List.cons(head, List.cons(next, List.cons(third, List.nil[Nat])))
        implies Nat.0 < cf.continuant
} by {
    if cf.coefficients = List.cons(head, List.cons(next, List.cons(third, List.nil[Nat]))) {
        continued_fraction_coefficients_valid(cf)
        finite_continued_fraction_coefficients(cf.coefficients)
        finite_continued_fraction_coefficients(
            List.cons(head, List.cons(next, List.cons(third, List.nil[Nat]))))
        continuant_triple_positive(head, next, third)
        continued_fraction_triple_continuant(cf, head, next, third)
        Nat.0 < cf.continuant
    }
}

/// A three-term continued fraction has nonzero continuant method.
theorem continued_fraction_triple_continuant_ne_zero(cf: ContinuedFraction,
    head: Nat, next: Nat, third: Nat) {
    cf.coefficients = List.cons(head, List.cons(next, List.cons(third, List.nil[Nat])))
        implies cf.continuant != Nat.0
} by {
    if cf.coefficients = List.cons(head, List.cons(next, List.cons(third, List.nil[Nat]))) {
        continued_fraction_triple_continuant_positive(cf, head, next, third)
        nat_positive_ne_zero(cf.continuant)
    }
}

/// A successor prefix value unfolds through the predecessor prefix of the tail.
theorem continued_fraction_prefix_value_cons_suc(
    head: Nat, tail: List[Nat], n: Nat
) {
    continued_fraction_prefix_value(List.cons(head, tail), n.suc) =
        Rat.from_nat(head) + continued_fraction_prefix_value(tail, n).inverse
} by {
    continued_fraction_prefix_coefficients_cons_suc(head, tail, n)
    continued_fraction_value_cons(head, continued_fraction_prefix_coefficients(tail, n))
}

/// A successor prefix continuant is the continuant of the displayed successor
/// prefix.
theorem continued_fraction_prefix_continuant_cons_suc(
    head: Nat, tail: List[Nat], n: Nat
) {
    continued_fraction_prefix_continuant(List.cons(head, tail), n.suc) =
        continuant(List.cons(head, continued_fraction_prefix_coefficients(tail, n)))
} by {
    continued_fraction_prefix_coefficients_cons_suc(head, tail, n)
}

/// A successor prefix numerator is the numerator of the displayed successor
/// prefix.
theorem continued_fraction_prefix_numerator_cons_suc(
    head: Nat, tail: List[Nat], n: Nat
) {
    continued_fraction_prefix_numerator(List.cons(head, tail), n.suc) =
        continued_fraction_numerator(
            List.cons(head, continued_fraction_prefix_coefficients(tail, n)))
} by {
    continued_fraction_prefix_coefficients_cons_suc(head, tail, n)
}

/// A successor prefix denominator is the denominator of the displayed
/// successor prefix.
theorem continued_fraction_prefix_denominator_cons_suc(
    head: Nat, tail: List[Nat], n: Nat
) {
    continued_fraction_prefix_denominator(List.cons(head, tail), n.suc) =
        continued_fraction_denominator(
            List.cons(head, continued_fraction_prefix_coefficients(tail, n)))
} by {
    continued_fraction_prefix_coefficients_cons_suc(head, tail, n)
}

/// A successor prefix convergent is the convergent of the displayed successor
/// prefix.
theorem continued_fraction_prefix_convergent_cons_suc(
    head: Nat, tail: List[Nat], n: Nat
) {
    continued_fraction_prefix_convergent(List.cons(head, tail), n.suc) =
        continued_fraction_convergent(
            List.cons(head, continued_fraction_prefix_coefficients(tail, n)))
} by {
    continued_fraction_prefix_coefficients_cons_suc(head, tail, n)
}

/// A valid prefix is a valid coefficient list.
theorem finite_continued_fraction_prefix_coefficients_elim(
    coefficients: List[Nat], n: Nat
) {
    finite_continued_fraction_prefix_coefficients(coefficients, n)
        implies finite_continued_fraction_coefficients(
            continued_fraction_prefix_coefficients(coefficients, n))
} by {
    if finite_continued_fraction_prefix_coefficients(coefficients, n) {
        finite_continued_fraction_prefix_coefficients(coefficients, n) =
            finite_continued_fraction_coefficients(
                continued_fraction_prefix_coefficients(coefficients, n))
        finite_continued_fraction_coefficients(
            continued_fraction_prefix_coefficients(coefficients, n))
    }
}

/// A valid coefficient-list prefix gives the prefix-validity predicate.
theorem finite_continued_fraction_prefix_coefficients_intro(
    coefficients: List[Nat], n: Nat
) {
    finite_continued_fraction_coefficients(
        continued_fraction_prefix_coefficients(coefficients, n))
        implies finite_continued_fraction_prefix_coefficients(coefficients, n)
} by {
    if finite_continued_fraction_coefficients(
        continued_fraction_prefix_coefficients(coefficients, n)) {
        finite_continued_fraction_prefix_coefficients(coefficients, n) =
            finite_continued_fraction_coefficients(
                continued_fraction_prefix_coefficients(coefficients, n))
        finite_continued_fraction_prefix_coefficients(coefficients, n)
    }
}

/// A positive prefix tail is a positive coefficient list.
theorem positive_continued_fraction_prefix_tail_elim(
    coefficients: List[Nat], n: Nat
) {
    positive_continued_fraction_prefix_tail(coefficients, n)
        implies positive_continued_fraction_tail(
            continued_fraction_prefix_coefficients(coefficients, n))
} by {
    if positive_continued_fraction_prefix_tail(coefficients, n) {
        positive_continued_fraction_prefix_tail(coefficients, n) =
            positive_continued_fraction_tail(
                continued_fraction_prefix_coefficients(coefficients, n))
        positive_continued_fraction_tail(
            continued_fraction_prefix_coefficients(coefficients, n))
    }
}

/// A positive coefficient-list prefix gives the positive-prefix-tail predicate.
theorem positive_continued_fraction_prefix_tail_intro(
    coefficients: List[Nat], n: Nat
) {
    positive_continued_fraction_tail(
        continued_fraction_prefix_coefficients(coefficients, n))
        implies positive_continued_fraction_prefix_tail(coefficients, n)
} by {
    if positive_continued_fraction_tail(
        continued_fraction_prefix_coefficients(coefficients, n)) {
        positive_continued_fraction_prefix_tail(coefficients, n) =
            positive_continued_fraction_tail(
                continued_fraction_prefix_coefficients(coefficients, n))
        positive_continued_fraction_prefix_tail(coefficients, n)
    }
}

/// The length-prefix of a valid coefficient list is valid.
theorem finite_continued_fraction_prefix_coefficients_length(
    coefficients: List[Nat]
) {
    finite_continued_fraction_coefficients(coefficients)
        implies finite_continued_fraction_prefix_coefficients(
            coefficients, coefficients.length)
} by {
    if finite_continued_fraction_coefficients(coefficients) {
        continued_fraction_prefix_coefficients_length(coefficients)
        finite_continued_fraction_coefficients(
            continued_fraction_prefix_coefficients(coefficients, coefficients.length))
        finite_continued_fraction_prefix_coefficients_intro(
            coefficients, coefficients.length)
    }
}

/// The length-prefix of a positive tail is a positive tail.
theorem positive_continued_fraction_prefix_tail_length(coefficients: List[Nat]) {
    positive_continued_fraction_tail(coefficients)
        implies positive_continued_fraction_prefix_tail(
            coefficients, coefficients.length)
} by {
    if positive_continued_fraction_tail(coefficients) {
        continued_fraction_prefix_coefficients_length(coefficients)
        positive_continued_fraction_tail(
            continued_fraction_prefix_coefficients(coefficients, coefficients.length))
        positive_continued_fraction_prefix_tail_intro(
            coefficients, coefficients.length)
    }
}

/// A continued fraction has valid full-length prefix.
theorem continued_fraction_has_valid_full_prefix(cf: ContinuedFraction) {
    cf.has_valid_prefix(cf.coefficients.length)
} by {
    continued_fraction_coefficients_valid(cf)
    finite_continued_fraction_prefix_coefficients_length(cf.coefficients)
    continued_fraction_has_valid_prefix_eq_coefficients_valid_prefix(
        cf, cf.coefficients.length)
}

/// A continued fraction whose whole coefficient list is positive has positive
/// full-length prefix tail.
theorem continued_fraction_has_positive_full_prefix_tail(
    cf: ContinuedFraction
) {
    positive_continued_fraction_tail(cf.coefficients)
        implies cf.has_positive_prefix_tail(cf.coefficients.length)
} by {
    if positive_continued_fraction_tail(cf.coefficients) {
        positive_continued_fraction_prefix_tail_length(cf.coefficients)
        continued_fraction_has_positive_prefix_tail_eq_coefficients_prefix_tail(
            cf, cf.coefficients.length)
    }
}

/// The length-prefix value is the value of the whole coefficient list.
theorem continued_fraction_prefix_value_length(coefficients: List[Nat]) {
    continued_fraction_prefix_value(coefficients, coefficients.length) =
        continued_fraction_value(coefficients)
} by {
    continued_fraction_prefix_coefficients_length(coefficients)
}

/// The length-prefix continuant is the continuant of the whole coefficient
/// list.
theorem continued_fraction_prefix_continuant_length(coefficients: List[Nat]) {
    continued_fraction_prefix_continuant(coefficients, coefficients.length) =
        continuant(coefficients)
} by {
    continued_fraction_prefix_coefficients_length(coefficients)
}

/// The length-prefix numerator is the numerator of the whole coefficient list.
theorem continued_fraction_prefix_numerator_length(coefficients: List[Nat]) {
    continued_fraction_prefix_numerator(coefficients, coefficients.length) =
        continued_fraction_numerator(coefficients)
} by {
    continued_fraction_prefix_coefficients_length(coefficients)
}

/// The length-prefix denominator is the denominator of the whole coefficient
/// list.
theorem continued_fraction_prefix_denominator_length(coefficients: List[Nat]) {
    continued_fraction_prefix_denominator(coefficients, coefficients.length) =
        continued_fraction_denominator(coefficients)
} by {
    continued_fraction_prefix_coefficients_length(coefficients)
}

/// The length-prefix convergent is the convergent of the whole coefficient
/// list.
theorem continued_fraction_prefix_convergent_length(coefficients: List[Nat]) {
    continued_fraction_prefix_convergent(coefficients, coefficients.length) =
        continued_fraction_convergent(coefficients)
} by {
    continued_fraction_prefix_coefficients_length(coefficients)
}

/// The full prefix value of a continued fraction is its value.
theorem continued_fraction_prefix_value_full(cf: ContinuedFraction) {
    cf.prefix_value(cf.coefficients.length) = cf.value
} by {
    continued_fraction_prefix_value_eq_coefficients_prefix_value(
        cf, cf.coefficients.length)
    continued_fraction_value_eq_coefficients_value(cf)
    continued_fraction_prefix_value_length(cf.coefficients)
}

/// The full prefix continuant of a continued fraction is its continuant.
theorem continued_fraction_prefix_continuant_full(cf: ContinuedFraction) {
    cf.prefix_continuant(cf.coefficients.length) = cf.continuant
} by {
    continued_fraction_prefix_continuant_eq_coefficients_prefix_continuant(
        cf, cf.coefficients.length)
    continued_fraction_continuant_eq_coefficients_continuant(cf)
    continued_fraction_prefix_continuant_length(cf.coefficients)
}

/// The full prefix numerator of a continued fraction is its numerator.
theorem continued_fraction_prefix_numerator_full(cf: ContinuedFraction) {
    cf.prefix_numerator(cf.coefficients.length) = cf.numerator
} by {
    continued_fraction_prefix_numerator_eq_coefficients_prefix_numerator(
        cf, cf.coefficients.length)
    continued_fraction_numerator_eq_coefficients_numerator(cf)
    continued_fraction_prefix_numerator_length(cf.coefficients)
}

/// The full prefix denominator of a continued fraction is its denominator.
theorem continued_fraction_prefix_denominator_full(cf: ContinuedFraction) {
    cf.prefix_denominator(cf.coefficients.length) = cf.denominator
} by {
    continued_fraction_prefix_denominator_eq_coefficients_prefix_denominator(
        cf, cf.coefficients.length)
    continued_fraction_denominator_eq_coefficients_denominator(cf)
    continued_fraction_prefix_denominator_length(cf.coefficients)
}

/// The full prefix convergent of a continued fraction is its convergent.
theorem continued_fraction_prefix_convergent_full(cf: ContinuedFraction) {
    cf.prefix_convergent(cf.coefficients.length) = cf.convergent
} by {
    continued_fraction_prefix_convergent_eq_coefficients_prefix_convergent(
        cf, cf.coefficients.length)
    continued_fraction_convergent_eq_coefficients_convergent(cf)
    continued_fraction_prefix_convergent_length(cf.coefficients)
}

/// A valid prefix has positive denominator.
theorem continued_fraction_prefix_denominator_positive_of_valid_coefficients(
    coefficients: List[Nat], n: Nat
) {
    finite_continued_fraction_prefix_coefficients(coefficients, n)
        implies Nat.0 < continued_fraction_prefix_denominator(coefficients, n)
} by {
    if finite_continued_fraction_prefix_coefficients(coefficients, n) {
        finite_continued_fraction_prefix_coefficients_elim(coefficients, n)
        continued_fraction_denominator_positive_of_valid_coefficients(
            continued_fraction_prefix_coefficients(coefficients, n))
        Nat.0 < continued_fraction_denominator(
            continued_fraction_prefix_coefficients(coefficients, n))
        Nat.0 < continued_fraction_prefix_denominator(coefficients, n)
    }
}

/// A valid prefix has nonzero denominator.
theorem continued_fraction_prefix_denominator_ne_zero_of_valid_coefficients(
    coefficients: List[Nat], n: Nat
) {
    finite_continued_fraction_prefix_coefficients(coefficients, n)
        implies continued_fraction_prefix_denominator(coefficients, n) != Nat.0
} by {
    if finite_continued_fraction_prefix_coefficients(coefficients, n) {
        continued_fraction_prefix_denominator_positive_of_valid_coefficients(
            coefficients, n)
        nat_positive_ne_zero(continued_fraction_prefix_denominator(coefficients, n))
    }
}

/// A valid prefix has positive convergent denominator.
theorem continued_fraction_prefix_convergent_second_positive_of_valid_coefficients(
    coefficients: List[Nat], n: Nat
) {
    finite_continued_fraction_prefix_coefficients(coefficients, n)
        implies Nat.0 < continued_fraction_prefix_convergent(coefficients, n).second
} by {
    if finite_continued_fraction_prefix_coefficients(coefficients, n) {
        finite_continued_fraction_prefix_coefficients_elim(coefficients, n)
        continued_fraction_convergent_second_positive_of_valid_coefficients(
            continued_fraction_prefix_coefficients(coefficients, n))
        Nat.0 < continued_fraction_convergent(
            continued_fraction_prefix_coefficients(coefficients, n)).second
        Nat.0 < continued_fraction_prefix_convergent(coefficients, n).second
    }
}

/// A valid prefix has nonzero convergent denominator.
theorem continued_fraction_prefix_convergent_second_ne_zero_of_valid_coefficients(
    coefficients: List[Nat], n: Nat
) {
    finite_continued_fraction_prefix_coefficients(coefficients, n)
        implies continued_fraction_prefix_convergent(coefficients, n).second != Nat.0
} by {
    if finite_continued_fraction_prefix_coefficients(coefficients, n) {
        continued_fraction_prefix_convergent_second_positive_of_valid_coefficients(
            coefficients, n)
        nat_positive_ne_zero(continued_fraction_prefix_convergent(coefficients, n).second)
    }
}

/// The first projection of a prefix convergent is the prefix numerator.
theorem continued_fraction_prefix_convergent_first_eq_numerator(
    coefficients: List[Nat], n: Nat
) {
    continued_fraction_prefix_convergent(coefficients, n).first =
        continued_fraction_prefix_numerator(coefficients, n)
} by {
    continued_fraction_convergent_first_eq_numerator(
        continued_fraction_prefix_coefficients(coefficients, n))
}

/// The prefix numerator is the first projection of the prefix convergent.
theorem continued_fraction_prefix_numerator_eq_convergent_first(
    coefficients: List[Nat], n: Nat
) {
    continued_fraction_prefix_numerator(coefficients, n) =
        continued_fraction_prefix_convergent(coefficients, n).first
} by {
    continued_fraction_prefix_convergent_first_eq_numerator(coefficients, n)
}

/// The second projection of a prefix convergent is the prefix denominator.
theorem continued_fraction_prefix_convergent_second_eq_denominator(
    coefficients: List[Nat], n: Nat
) {
    continued_fraction_prefix_convergent(coefficients, n).second =
        continued_fraction_prefix_denominator(coefficients, n)
} by {
    continued_fraction_convergent_second_eq_denominator(
        continued_fraction_prefix_coefficients(coefficients, n))
}

/// The prefix denominator is the second projection of the prefix convergent.
theorem continued_fraction_prefix_denominator_eq_convergent_second(
    coefficients: List[Nat], n: Nat
) {
    continued_fraction_prefix_denominator(coefficients, n) =
        continued_fraction_prefix_convergent(coefficients, n).second
} by {
    continued_fraction_prefix_convergent_second_eq_denominator(coefficients, n)
}

/// A prefix convergent is the pair formed by its prefix numerator and
/// denominator.
theorem continued_fraction_prefix_convergent_eq_numerator_denominator_pair(
    coefficients: List[Nat], n: Nat
) {
    continued_fraction_prefix_convergent(coefficients, n) =
        Pair.new(continued_fraction_prefix_numerator(coefficients, n),
            continued_fraction_prefix_denominator(coefficients, n))
} by {
    continued_fraction_prefix_convergent_first_eq_numerator(coefficients, n)
    continued_fraction_prefix_convergent_second_eq_denominator(coefficients, n)
    pair_new_first(continued_fraction_prefix_numerator(coefficients, n),
        continued_fraction_prefix_denominator(coefficients, n))
    pair_new_second(continued_fraction_prefix_numerator(coefficients, n),
        continued_fraction_prefix_denominator(coefficients, n))
    pair_ext(continued_fraction_prefix_convergent(coefficients, n),
        Pair.new(continued_fraction_prefix_numerator(coefficients, n),
            continued_fraction_prefix_denominator(coefficients, n)))
}

/// A valid prefix has numerator equal to prefix continuant.
theorem continued_fraction_prefix_numerator_eq_continuant_of_valid_coefficients(
    coefficients: List[Nat], n: Nat
) {
    finite_continued_fraction_prefix_coefficients(coefficients, n)
        implies continued_fraction_prefix_numerator(coefficients, n) =
            continued_fraction_prefix_continuant(coefficients, n)
} by {
    if finite_continued_fraction_prefix_coefficients(coefficients, n) {
        finite_continued_fraction_prefix_coefficients_elim(coefficients, n)
        continued_fraction_numerator_eq_continuant_of_valid_coefficients(
            continued_fraction_prefix_coefficients(coefficients, n))
        continued_fraction_numerator(continued_fraction_prefix_coefficients(coefficients, n)) =
            continuant(continued_fraction_prefix_coefficients(coefficients, n))
        continued_fraction_prefix_numerator(coefficients, n) =
            continued_fraction_prefix_continuant(coefficients, n)
    }
}

/// A valid prefix has continuant equal to prefix numerator.
theorem continued_fraction_prefix_continuant_eq_numerator_of_valid_coefficients(
    coefficients: List[Nat], n: Nat
) {
    finite_continued_fraction_prefix_coefficients(coefficients, n)
        implies continued_fraction_prefix_continuant(coefficients, n) =
            continued_fraction_prefix_numerator(coefficients, n)
} by {
    if finite_continued_fraction_prefix_coefficients(coefficients, n) {
        continued_fraction_prefix_numerator_eq_continuant_of_valid_coefficients(
            coefficients, n)
    }
}

/// A valid prefix has convergent first projection equal to prefix continuant.
theorem continued_fraction_prefix_convergent_first_eq_continuant_of_valid_coefficients(
    coefficients: List[Nat], n: Nat
) {
    finite_continued_fraction_prefix_coefficients(coefficients, n)
        implies continued_fraction_prefix_convergent(coefficients, n).first =
            continued_fraction_prefix_continuant(coefficients, n)
} by {
    if finite_continued_fraction_prefix_coefficients(coefficients, n) {
        continued_fraction_prefix_convergent_first_eq_numerator(coefficients, n)
        continued_fraction_prefix_numerator_eq_continuant_of_valid_coefficients(
            coefficients, n)
        continued_fraction_prefix_convergent(coefficients, n).first =
            continued_fraction_prefix_continuant(coefficients, n)
    }
}

/// A valid prefix has continuant equal to the first projection of its
/// convergent.
theorem continued_fraction_prefix_continuant_eq_convergent_first_of_valid_coefficients(
    coefficients: List[Nat], n: Nat
) {
    finite_continued_fraction_prefix_coefficients(coefficients, n)
        implies continued_fraction_prefix_continuant(coefficients, n) =
            continued_fraction_prefix_convergent(coefficients, n).first
} by {
    if finite_continued_fraction_prefix_coefficients(coefficients, n) {
        continued_fraction_prefix_convergent_first_eq_continuant_of_valid_coefficients(
            coefficients, n)
    }
}

/// A valid prefix convergent is the pair formed by the prefix continuant and
/// denominator.
theorem continued_fraction_prefix_convergent_eq_continuant_denominator_pair_of_valid_coefficients(
    coefficients: List[Nat], n: Nat
) {
    finite_continued_fraction_prefix_coefficients(coefficients, n)
        implies continued_fraction_prefix_convergent(coefficients, n) =
            Pair.new(continued_fraction_prefix_continuant(coefficients, n),
                continued_fraction_prefix_denominator(coefficients, n))
} by {
    if finite_continued_fraction_prefix_coefficients(coefficients, n) {
        continued_fraction_prefix_convergent_eq_numerator_denominator_pair(
            coefficients, n)
        continued_fraction_prefix_convergent(coefficients, n) =
            Pair.new(continued_fraction_prefix_numerator(coefficients, n),
                continued_fraction_prefix_denominator(coefficients, n))
        continued_fraction_prefix_numerator_eq_continuant_of_valid_coefficients(
            coefficients, n)
        continued_fraction_prefix_numerator(coefficients, n) =
            continued_fraction_prefix_continuant(coefficients, n)
        Pair.new(continued_fraction_prefix_numerator(coefficients, n),
            continued_fraction_prefix_denominator(coefficients, n)) =
            Pair.new(continued_fraction_prefix_continuant(coefficients, n),
                continued_fraction_prefix_denominator(coefficients, n))
        continued_fraction_prefix_convergent(coefficients, n) =
            Pair.new(continued_fraction_prefix_continuant(coefficients, n),
                continued_fraction_prefix_denominator(coefficients, n))
    }
}

/// A valid positive prefix has positive numerator.
theorem continued_fraction_prefix_numerator_positive_of_valid_positive_coefficients(
    coefficients: List[Nat], n: Nat
) {
    finite_continued_fraction_prefix_coefficients(coefficients, n)
        and positive_continued_fraction_prefix_tail(coefficients, n)
        implies Nat.0 < continued_fraction_prefix_numerator(coefficients, n)
} by {
    if finite_continued_fraction_prefix_coefficients(coefficients, n)
        and positive_continued_fraction_prefix_tail(coefficients, n) {
        finite_continued_fraction_prefix_coefficients_elim(coefficients, n)
        positive_continued_fraction_prefix_tail_elim(coefficients, n)
        finite_continued_fraction_coefficients(
            continued_fraction_prefix_coefficients(coefficients, n))
        positive_continued_fraction_tail(
            continued_fraction_prefix_coefficients(coefficients, n))
        finite_continued_fraction_coefficients(continued_fraction_prefix_coefficients(coefficients, n)) and positive_continued_fraction_tail(continued_fraction_prefix_coefficients(coefficients, n))
        continued_fraction_numerator_positive_of_valid_positive_coefficients(
            continued_fraction_prefix_coefficients(coefficients, n))
        Nat.0 < continued_fraction_numerator(
            continued_fraction_prefix_coefficients(coefficients, n))
        Nat.0 < continued_fraction_prefix_numerator(coefficients, n)
    }
}

/// A valid positive prefix has nonzero numerator.
theorem continued_fraction_prefix_numerator_ne_zero_of_valid_positive_coefficients(
    coefficients: List[Nat], n: Nat
) {
    finite_continued_fraction_prefix_coefficients(coefficients, n)
        and positive_continued_fraction_prefix_tail(coefficients, n)
        implies continued_fraction_prefix_numerator(coefficients, n) != Nat.0
} by {
    if finite_continued_fraction_prefix_coefficients(coefficients, n)
        and positive_continued_fraction_prefix_tail(coefficients, n) {
        continued_fraction_prefix_numerator_positive_of_valid_positive_coefficients(
            coefficients, n)
        nat_positive_ne_zero(continued_fraction_prefix_numerator(coefficients, n))
    }
}

/// A valid positive prefix has positive continuant.
theorem continued_fraction_prefix_continuant_positive_of_valid_positive_coefficients(
    coefficients: List[Nat], n: Nat
) {
    finite_continued_fraction_prefix_coefficients(coefficients, n)
        and positive_continued_fraction_prefix_tail(coefficients, n)
        implies Nat.0 < continued_fraction_prefix_continuant(coefficients, n)
} by {
    if finite_continued_fraction_prefix_coefficients(coefficients, n)
        and positive_continued_fraction_prefix_tail(coefficients, n) {
        continued_fraction_prefix_numerator_positive_of_valid_positive_coefficients(
            coefficients, n)
        Nat.0 < continued_fraction_prefix_numerator(coefficients, n)
        continued_fraction_prefix_numerator_eq_continuant_of_valid_coefficients(
            coefficients, n)
        Nat.0 < continued_fraction_prefix_continuant(coefficients, n)
    }
}

/// A valid positive prefix has nonzero continuant.
theorem continued_fraction_prefix_continuant_ne_zero_of_valid_positive_coefficients(
    coefficients: List[Nat], n: Nat
) {
    finite_continued_fraction_prefix_coefficients(coefficients, n)
        and positive_continued_fraction_prefix_tail(coefficients, n)
        implies continued_fraction_prefix_continuant(coefficients, n) != Nat.0
} by {
    if finite_continued_fraction_prefix_coefficients(coefficients, n)
        and positive_continued_fraction_prefix_tail(coefficients, n) {
        continued_fraction_prefix_continuant_positive_of_valid_positive_coefficients(
            coefficients, n)
        nat_positive_ne_zero(continued_fraction_prefix_continuant(coefficients, n))
    }
}

/// A valid positive prefix has positive convergent numerator projection.
theorem continued_fraction_prefix_convergent_first_positive_of_valid_positive_coefficients(
    coefficients: List[Nat], n: Nat
) {
    finite_continued_fraction_prefix_coefficients(coefficients, n)
        and positive_continued_fraction_prefix_tail(coefficients, n)
        implies Nat.0 < continued_fraction_prefix_convergent(coefficients, n).first
} by {
    if finite_continued_fraction_prefix_coefficients(coefficients, n)
        and positive_continued_fraction_prefix_tail(coefficients, n) {
        continued_fraction_prefix_convergent_first_eq_numerator(coefficients, n)
        continued_fraction_prefix_numerator_positive_of_valid_positive_coefficients(
            coefficients, n)
        Nat.0 < continued_fraction_prefix_numerator(coefficients, n)
        Nat.0 < continued_fraction_prefix_convergent(coefficients, n).first
    }
}

/// A valid positive prefix has nonzero convergent numerator projection.
theorem continued_fraction_prefix_convergent_first_ne_zero_of_valid_positive_coefficients(
    coefficients: List[Nat], n: Nat
) {
    finite_continued_fraction_prefix_coefficients(coefficients, n)
        and positive_continued_fraction_prefix_tail(coefficients, n)
        implies continued_fraction_prefix_convergent(coefficients, n).first != Nat.0
} by {
    if finite_continued_fraction_prefix_coefficients(coefficients, n)
        and positive_continued_fraction_prefix_tail(coefficients, n) {
        continued_fraction_prefix_convergent_first_positive_of_valid_positive_coefficients(
            coefficients, n)
        nat_positive_ne_zero(continued_fraction_prefix_convergent(coefficients, n).first)
    }
}

/// A valid prefix method gives a valid prefix coefficient list.
theorem continued_fraction_has_valid_prefix_elim(cf: ContinuedFraction, n: Nat) {
    cf.has_valid_prefix(n)
        implies finite_continued_fraction_prefix_coefficients(cf.coefficients, n)
} by {
    if cf.has_valid_prefix(n) {
        continued_fraction_has_valid_prefix_eq_coefficients_valid_prefix(cf, n)
        finite_continued_fraction_prefix_coefficients(cf.coefficients, n)
    }
}

/// A valid prefix coefficient list gives the valid prefix method.
theorem continued_fraction_has_valid_prefix_intro(cf: ContinuedFraction, n: Nat) {
    finite_continued_fraction_prefix_coefficients(cf.coefficients, n)
        implies cf.has_valid_prefix(n)
} by {
    if finite_continued_fraction_prefix_coefficients(cf.coefficients, n) {
        continued_fraction_has_valid_prefix_eq_coefficients_valid_prefix(cf, n)
        cf.has_valid_prefix(n)
    }
}

/// A positive-prefix-tail method gives a positive prefix coefficient list.
theorem continued_fraction_has_positive_prefix_tail_elim(
    cf: ContinuedFraction, n: Nat
) {
    cf.has_positive_prefix_tail(n)
        implies positive_continued_fraction_prefix_tail(cf.coefficients, n)
} by {
    if cf.has_positive_prefix_tail(n) {
        continued_fraction_has_positive_prefix_tail_eq_coefficients_prefix_tail(cf, n)
        positive_continued_fraction_prefix_tail(cf.coefficients, n)
    }
}

/// A positive prefix coefficient list gives the positive-prefix-tail method.
theorem continued_fraction_has_positive_prefix_tail_intro(
    cf: ContinuedFraction, n: Nat
) {
    positive_continued_fraction_prefix_tail(cf.coefficients, n)
        implies cf.has_positive_prefix_tail(n)
} by {
    if positive_continued_fraction_prefix_tail(cf.coefficients, n) {
        continued_fraction_has_positive_prefix_tail_eq_coefficients_prefix_tail(cf, n)
        cf.has_positive_prefix_tail(n)
    }
}

/// A valid prefix of a continued fraction has positive denominator.
theorem continued_fraction_prefix_denominator_positive(
    cf: ContinuedFraction, n: Nat
) {
    cf.has_valid_prefix(n) implies Nat.0 < cf.prefix_denominator(n)
} by {
    if cf.has_valid_prefix(n) {
        continued_fraction_has_valid_prefix_elim(cf, n)
        continued_fraction_prefix_denominator_positive_of_valid_coefficients(
            cf.coefficients, n)
        Nat.0 < continued_fraction_prefix_denominator(cf.coefficients, n)
        continued_fraction_prefix_denominator_eq_coefficients_prefix_denominator(cf, n)
        Nat.0 < cf.prefix_denominator(n)
    }
}

/// A valid prefix of a continued fraction has nonzero denominator.
theorem continued_fraction_prefix_denominator_ne_zero(
    cf: ContinuedFraction, n: Nat
) {
    cf.has_valid_prefix(n) implies cf.prefix_denominator(n) != Nat.0
} by {
    if cf.has_valid_prefix(n) {
        continued_fraction_prefix_denominator_positive(cf, n)
        nat_positive_ne_zero(cf.prefix_denominator(n))
    }
}

/// A valid prefix of a continued fraction has positive convergent denominator.
theorem continued_fraction_prefix_convergent_second_positive(
    cf: ContinuedFraction, n: Nat
) {
    cf.has_valid_prefix(n) implies Nat.0 < cf.prefix_convergent(n).second
} by {
    if cf.has_valid_prefix(n) {
        continued_fraction_has_valid_prefix_elim(cf, n)
        continued_fraction_prefix_convergent_second_positive_of_valid_coefficients(
            cf.coefficients, n)
        Nat.0 < continued_fraction_prefix_convergent(cf.coefficients, n).second
        continued_fraction_prefix_convergent_eq_coefficients_prefix_convergent(cf, n)
        Nat.0 < cf.prefix_convergent(n).second
    }
}

/// A valid prefix of a continued fraction has nonzero convergent denominator.
theorem continued_fraction_prefix_convergent_second_ne_zero(
    cf: ContinuedFraction, n: Nat
) {
    cf.has_valid_prefix(n) implies cf.prefix_convergent(n).second != Nat.0
} by {
    if cf.has_valid_prefix(n) {
        continued_fraction_prefix_convergent_second_positive(cf, n)
        nat_positive_ne_zero(cf.prefix_convergent(n).second)
    }
}

/// The first projection of a continued-fraction prefix convergent is its
/// prefix numerator.
theorem continued_fraction_prefix_convergent_first_eq_prefix_numerator(
    cf: ContinuedFraction, n: Nat
) {
    cf.prefix_convergent(n).first = cf.prefix_numerator(n)
} by {
    continued_fraction_prefix_convergent_eq_coefficients_prefix_convergent(cf, n)
    continued_fraction_prefix_numerator_eq_coefficients_prefix_numerator(cf, n)
    continued_fraction_prefix_convergent_first_eq_numerator(cf.coefficients, n)
}

/// The prefix numerator of a continued fraction is the first projection of its
/// prefix convergent.
theorem continued_fraction_prefix_numerator_eq_prefix_convergent_first(
    cf: ContinuedFraction, n: Nat
) {
    cf.prefix_numerator(n) = cf.prefix_convergent(n).first
} by {
    continued_fraction_prefix_convergent_first_eq_prefix_numerator(cf, n)
}

/// The second projection of a continued-fraction prefix convergent is its
/// prefix denominator.
theorem continued_fraction_prefix_convergent_second_eq_prefix_denominator(
    cf: ContinuedFraction, n: Nat
) {
    cf.prefix_convergent(n).second = cf.prefix_denominator(n)
} by {
    continued_fraction_prefix_convergent_eq_coefficients_prefix_convergent(cf, n)
    continued_fraction_prefix_denominator_eq_coefficients_prefix_denominator(cf, n)
    continued_fraction_prefix_convergent_second_eq_denominator(cf.coefficients, n)
}

/// The prefix denominator of a continued fraction is the second projection of
/// its prefix convergent.
theorem continued_fraction_prefix_denominator_eq_prefix_convergent_second(
    cf: ContinuedFraction, n: Nat
) {
    cf.prefix_denominator(n) = cf.prefix_convergent(n).second
} by {
    continued_fraction_prefix_convergent_second_eq_prefix_denominator(cf, n)
}

/// A continued-fraction prefix convergent is the pair formed by its prefix
/// numerator and denominator.
theorem continued_fraction_prefix_convergent_eq_prefix_numerator_denominator_pair(
    cf: ContinuedFraction, n: Nat
) {
    cf.prefix_convergent(n) = Pair.new(cf.prefix_numerator(n), cf.prefix_denominator(n))
} by {
    continued_fraction_prefix_convergent_first_eq_prefix_numerator(cf, n)
    continued_fraction_prefix_convergent_second_eq_prefix_denominator(cf, n)
    pair_new_first(cf.prefix_numerator(n), cf.prefix_denominator(n))
    pair_new_second(cf.prefix_numerator(n), cf.prefix_denominator(n))
    pair_ext(cf.prefix_convergent(n), Pair.new(cf.prefix_numerator(n),
        cf.prefix_denominator(n)))
}

/// A valid prefix has prefix numerator equal to prefix continuant.
theorem continued_fraction_prefix_numerator_eq_prefix_continuant_of_valid_prefix(
    cf: ContinuedFraction, n: Nat
) {
    cf.has_valid_prefix(n) implies cf.prefix_numerator(n) = cf.prefix_continuant(n)
} by {
    if cf.has_valid_prefix(n) {
        continued_fraction_has_valid_prefix_elim(cf, n)
        continued_fraction_prefix_numerator_eq_continuant_of_valid_coefficients(
            cf.coefficients, n)
        continued_fraction_prefix_numerator(cf.coefficients, n) =
            continued_fraction_prefix_continuant(cf.coefficients, n)
        continued_fraction_prefix_numerator_eq_coefficients_prefix_numerator(cf, n)
        continued_fraction_prefix_continuant_eq_coefficients_prefix_continuant(cf, n)
        cf.prefix_numerator(n) = cf.prefix_continuant(n)
    }
}

/// A valid prefix has prefix continuant equal to prefix numerator.
theorem continued_fraction_prefix_continuant_eq_prefix_numerator_of_valid_prefix(
    cf: ContinuedFraction, n: Nat
) {
    cf.has_valid_prefix(n) implies cf.prefix_continuant(n) = cf.prefix_numerator(n)
} by {
    if cf.has_valid_prefix(n) {
        continued_fraction_prefix_numerator_eq_prefix_continuant_of_valid_prefix(
            cf, n)
    }
}

/// A valid prefix has convergent first projection equal to prefix continuant.
theorem continued_fraction_prefix_convergent_first_eq_prefix_continuant_of_valid_prefix(
    cf: ContinuedFraction, n: Nat
) {
    cf.has_valid_prefix(n)
        implies cf.prefix_convergent(n).first = cf.prefix_continuant(n)
} by {
    if cf.has_valid_prefix(n) {
        continued_fraction_prefix_convergent_first_eq_prefix_numerator(cf, n)
        continued_fraction_prefix_numerator_eq_prefix_continuant_of_valid_prefix(
            cf, n)
        cf.prefix_convergent(n).first = cf.prefix_continuant(n)
    }
}

/// A valid prefix has prefix continuant equal to the first projection of its
/// prefix convergent.
theorem continued_fraction_prefix_continuant_eq_prefix_convergent_first_of_valid_prefix(
    cf: ContinuedFraction, n: Nat
) {
    cf.has_valid_prefix(n)
        implies cf.prefix_continuant(n) = cf.prefix_convergent(n).first
} by {
    if cf.has_valid_prefix(n) {
        continued_fraction_prefix_convergent_first_eq_prefix_continuant_of_valid_prefix(
            cf, n)
    }
}

/// A valid prefix convergent is the pair formed by prefix continuant and
/// denominator.
theorem continued_fraction_prefix_convergent_eq_prefix_continuant_denominator_pair_of_valid_prefix(
    cf: ContinuedFraction, n: Nat
) {
    cf.has_valid_prefix(n)
        implies cf.prefix_convergent(n) =
            Pair.new(cf.prefix_continuant(n), cf.prefix_denominator(n))
} by {
    if cf.has_valid_prefix(n) {
        continued_fraction_prefix_convergent_eq_prefix_numerator_denominator_pair(cf, n)
        cf.prefix_convergent(n) =
            Pair.new(cf.prefix_numerator(n), cf.prefix_denominator(n))
        continued_fraction_prefix_numerator_eq_prefix_continuant_of_valid_prefix(
            cf, n)
        cf.prefix_numerator(n) = cf.prefix_continuant(n)
        Pair.new(cf.prefix_numerator(n), cf.prefix_denominator(n)) =
            Pair.new(cf.prefix_continuant(n), cf.prefix_denominator(n))
        cf.prefix_convergent(n) =
            Pair.new(cf.prefix_continuant(n), cf.prefix_denominator(n))
    }
}

/// A valid positive prefix of a continued fraction has positive numerator.
theorem continued_fraction_prefix_numerator_positive_of_valid_positive_prefix(
    cf: ContinuedFraction, n: Nat
) {
    cf.has_valid_prefix(n) and cf.has_positive_prefix_tail(n)
        implies Nat.0 < cf.prefix_numerator(n)
} by {
    if cf.has_valid_prefix(n) and cf.has_positive_prefix_tail(n) {
        continued_fraction_has_valid_prefix_elim(cf, n)
        continued_fraction_has_positive_prefix_tail_elim(cf, n)
        finite_continued_fraction_prefix_coefficients(cf.coefficients, n)
        positive_continued_fraction_prefix_tail(cf.coefficients, n)
        finite_continued_fraction_prefix_coefficients(cf.coefficients, n) and positive_continued_fraction_prefix_tail(cf.coefficients, n)
        continued_fraction_prefix_numerator_positive_of_valid_positive_coefficients(
            cf.coefficients, n)
        Nat.0 < continued_fraction_prefix_numerator(cf.coefficients, n)
        continued_fraction_prefix_numerator_eq_coefficients_prefix_numerator(cf, n)
        Nat.0 < cf.prefix_numerator(n)
    }
}

/// A valid positive prefix of a continued fraction has nonzero numerator.
theorem continued_fraction_prefix_numerator_ne_zero_of_valid_positive_prefix(
    cf: ContinuedFraction, n: Nat
) {
    cf.has_valid_prefix(n) and cf.has_positive_prefix_tail(n)
        implies cf.prefix_numerator(n) != Nat.0
} by {
    if cf.has_valid_prefix(n) and cf.has_positive_prefix_tail(n) {
        continued_fraction_prefix_numerator_positive_of_valid_positive_prefix(cf, n)
        nat_positive_ne_zero(cf.prefix_numerator(n))
    }
}

/// A valid positive prefix of a continued fraction has positive continuant.
theorem continued_fraction_prefix_continuant_positive_of_valid_positive_prefix(
    cf: ContinuedFraction, n: Nat
) {
    cf.has_valid_prefix(n) and cf.has_positive_prefix_tail(n)
        implies Nat.0 < cf.prefix_continuant(n)
} by {
    if cf.has_valid_prefix(n) and cf.has_positive_prefix_tail(n) {
        continued_fraction_prefix_numerator_positive_of_valid_positive_prefix(cf, n)
        Nat.0 < cf.prefix_numerator(n)
        continued_fraction_prefix_numerator_eq_prefix_continuant_of_valid_prefix(
            cf, n)
        cf.prefix_numerator(n) = cf.prefix_continuant(n)
        Nat.0 < cf.prefix_continuant(n)
    }
}

/// A valid positive prefix of a continued fraction has nonzero continuant.
theorem continued_fraction_prefix_continuant_ne_zero_of_valid_positive_prefix(
    cf: ContinuedFraction, n: Nat
) {
    cf.has_valid_prefix(n) and cf.has_positive_prefix_tail(n)
        implies cf.prefix_continuant(n) != Nat.0
} by {
    if cf.has_valid_prefix(n) and cf.has_positive_prefix_tail(n) {
        continued_fraction_prefix_continuant_positive_of_valid_positive_prefix(cf, n)
        nat_positive_ne_zero(cf.prefix_continuant(n))
    }
}

/// A valid positive prefix of a continued fraction has positive convergent
/// numerator projection.
theorem continued_fraction_prefix_convergent_first_positive_of_valid_positive_prefix(
    cf: ContinuedFraction, n: Nat
) {
    cf.has_valid_prefix(n) and cf.has_positive_prefix_tail(n)
        implies Nat.0 < cf.prefix_convergent(n).first
} by {
    if cf.has_valid_prefix(n) and cf.has_positive_prefix_tail(n) {
        continued_fraction_prefix_convergent_first_eq_prefix_numerator(cf, n)
        continued_fraction_prefix_numerator_positive_of_valid_positive_prefix(cf, n)
        Nat.0 < cf.prefix_numerator(n)
        Nat.0 < cf.prefix_convergent(n).first
    }
}

/// A valid positive prefix of a continued fraction has nonzero convergent
/// numerator projection.
theorem continued_fraction_prefix_convergent_first_ne_zero_of_valid_positive_prefix(
    cf: ContinuedFraction, n: Nat
) {
    cf.has_valid_prefix(n) and cf.has_positive_prefix_tail(n)
        implies cf.prefix_convergent(n).first != Nat.0
} by {
    if cf.has_valid_prefix(n) and cf.has_positive_prefix_tail(n) {
        continued_fraction_prefix_convergent_first_positive_of_valid_positive_prefix(
            cf, n)
        nat_positive_ne_zero(cf.prefix_convergent(n).first)
    }
}

/// The full prefix of a continued fraction has positive denominator.
theorem continued_fraction_full_prefix_denominator_positive(cf: ContinuedFraction) {
    Nat.0 < cf.prefix_denominator(cf.coefficients.length)
} by {
    continued_fraction_has_valid_full_prefix(cf)
    continued_fraction_prefix_denominator_positive(cf, cf.coefficients.length)
}

/// The full prefix of a continued fraction has nonzero denominator.
theorem continued_fraction_full_prefix_denominator_ne_zero(cf: ContinuedFraction) {
    cf.prefix_denominator(cf.coefficients.length) != Nat.0
} by {
    continued_fraction_full_prefix_denominator_positive(cf)
    nat_positive_ne_zero(cf.prefix_denominator(cf.coefficients.length))
}

/// The full prefix of a continued fraction has positive convergent denominator.
theorem continued_fraction_full_prefix_convergent_second_positive(
    cf: ContinuedFraction
) {
    Nat.0 < cf.prefix_convergent(cf.coefficients.length).second
} by {
    continued_fraction_has_valid_full_prefix(cf)
    continued_fraction_prefix_convergent_second_positive(
        cf, cf.coefficients.length)
}

/// The full prefix of a continued fraction has nonzero convergent denominator.
theorem continued_fraction_full_prefix_convergent_second_ne_zero(
    cf: ContinuedFraction
) {
    cf.prefix_convergent(cf.coefficients.length).second != Nat.0
} by {
    continued_fraction_full_prefix_convergent_second_positive(cf)
    nat_positive_ne_zero(cf.prefix_convergent(cf.coefficients.length).second)
}

/// The zero prefix of a continued fraction has empty coefficients.
theorem continued_fraction_prefix_coefficients_zero_method(cf: ContinuedFraction) {
    cf.prefix_coefficients(Nat.0) = List.nil[Nat]
} by {
    continued_fraction_prefix_coefficients_eq_coefficients_prefix(cf, Nat.0)
    continued_fraction_prefix_coefficients_zero(cf.coefficients)
}

/// The zero prefix of a continued fraction is not a finite simple continued
/// fraction.
theorem continued_fraction_has_valid_prefix_zero_false(cf: ContinuedFraction) {
    not cf.has_valid_prefix(Nat.0)
} by {
    continued_fraction_has_valid_prefix_eq_coefficients_valid_prefix(cf, Nat.0)
    finite_continued_fraction_prefix_coefficients_zero(cf.coefficients)
}

/// The zero prefix of a continued fraction has positive tail.
theorem continued_fraction_has_positive_prefix_tail_zero(cf: ContinuedFraction) {
    cf.has_positive_prefix_tail(Nat.0)
} by {
    positive_continued_fraction_prefix_tail_zero(cf.coefficients)
    continued_fraction_has_positive_prefix_tail_eq_coefficients_prefix_tail(
        cf, Nat.0)
}

/// The zero prefix of a continued fraction has value zero.
theorem continued_fraction_prefix_value_zero_method(cf: ContinuedFraction) {
    cf.prefix_value(Nat.0) = Rat.0
} by {
    continued_fraction_prefix_value_eq_coefficients_prefix_value(cf, Nat.0)
    continued_fraction_prefix_value_zero(cf.coefficients)
}

/// The zero prefix of a continued fraction has continuant one.
theorem continued_fraction_prefix_continuant_zero_method(cf: ContinuedFraction) {
    cf.prefix_continuant(Nat.0) = Nat.1
} by {
    continued_fraction_prefix_continuant_eq_coefficients_prefix_continuant(cf, Nat.0)
    continued_fraction_prefix_continuant_zero(cf.coefficients)
}

/// The zero prefix of a continued fraction has numerator zero.
theorem continued_fraction_prefix_numerator_zero_method(cf: ContinuedFraction) {
    cf.prefix_numerator(Nat.0) = Nat.0
} by {
    continued_fraction_prefix_numerator_eq_coefficients_prefix_numerator(cf, Nat.0)
    continued_fraction_prefix_numerator_zero(cf.coefficients)
}

/// The zero prefix of a continued fraction has denominator one.
theorem continued_fraction_prefix_denominator_zero_method(cf: ContinuedFraction) {
    cf.prefix_denominator(Nat.0) = Nat.1
} by {
    continued_fraction_prefix_denominator_eq_coefficients_prefix_denominator(cf, Nat.0)
    continued_fraction_prefix_denominator_zero(cf.coefficients)
}

/// The zero prefix of a continued fraction has the conventional empty
/// convergent.
theorem continued_fraction_prefix_convergent_zero_method(cf: ContinuedFraction) {
    cf.prefix_convergent(Nat.0) = Pair.new(Nat.0, Nat.1)
} by {
    continued_fraction_prefix_convergent_eq_coefficients_prefix_convergent(cf, Nat.0)
    continued_fraction_prefix_convergent_zero(cf.coefficients)
}

/// The zero prefix of a continued fraction has positive denominator.
theorem continued_fraction_prefix_denominator_zero_positive(cf: ContinuedFraction) {
    Nat.0 < cf.prefix_denominator(Nat.0)
} by {
    continued_fraction_prefix_denominator_zero_method(cf)
    Nat.0 < Nat.1
}

/// The zero prefix of a continued fraction has nonzero denominator.
theorem continued_fraction_prefix_denominator_zero_ne_zero(cf: ContinuedFraction) {
    cf.prefix_denominator(Nat.0) != Nat.0
} by {
    continued_fraction_prefix_denominator_zero_positive(cf)
    nat_positive_ne_zero(cf.prefix_denominator(Nat.0))
}

/// The zero prefix of a continued fraction has positive convergent denominator.
theorem continued_fraction_prefix_convergent_zero_second_positive(
    cf: ContinuedFraction
) {
    Nat.0 < cf.prefix_convergent(Nat.0).second
} by {
    continued_fraction_prefix_convergent_zero_method(cf)
    pair_new_second(Nat.0, Nat.1)
    Nat.0 < Nat.1
}

/// The zero prefix of a continued fraction has nonzero convergent denominator.
theorem continued_fraction_prefix_convergent_zero_second_ne_zero(
    cf: ContinuedFraction
) {
    cf.prefix_convergent(Nat.0).second != Nat.0
} by {
    continued_fraction_prefix_convergent_zero_second_positive(cf)
    nat_positive_ne_zero(cf.prefix_convergent(Nat.0).second)
}

/// A displayed nonempty continued fraction has successor prefix coefficients
/// formed by consing the displayed head onto the predecessor tail prefix.
theorem continued_fraction_prefix_coefficients_cons_suc_method(
    cf: ContinuedFraction, head: Nat, tail: List[Nat], n: Nat
) {
    cf.coefficients = List.cons(head, tail)
        implies cf.prefix_coefficients(n.suc) =
            List.cons(head, continued_fraction_prefix_coefficients(tail, n))
} by {
    if cf.coefficients = List.cons(head, tail) {
        continued_fraction_prefix_coefficients_eq_coefficients_prefix(cf, n.suc)
        continued_fraction_prefix_coefficients_cons_suc(head, tail, n)
    }
}

/// A displayed nonempty continued fraction has successor prefix value obtained
/// from the displayed head and predecessor tail prefix.
theorem continued_fraction_prefix_value_cons_suc_method(
    cf: ContinuedFraction, head: Nat, tail: List[Nat], n: Nat
) {
    cf.coefficients = List.cons(head, tail)
        implies cf.prefix_value(n.suc) =
            Rat.from_nat(head) + continued_fraction_prefix_value(tail, n).inverse
} by {
    if cf.coefficients = List.cons(head, tail) {
        continued_fraction_prefix_value_eq_coefficients_prefix_value(cf, n.suc)
        continued_fraction_prefix_value_cons_suc(head, tail, n)
    }
}

/// A displayed nonempty continued fraction has successor prefix continuant
/// equal to the displayed successor prefix continuant.
theorem continued_fraction_prefix_continuant_cons_suc_method(
    cf: ContinuedFraction, head: Nat, tail: List[Nat], n: Nat
) {
    cf.coefficients = List.cons(head, tail)
        implies cf.prefix_continuant(n.suc) =
            continuant(List.cons(head, continued_fraction_prefix_coefficients(tail, n)))
} by {
    if cf.coefficients = List.cons(head, tail) {
        continued_fraction_prefix_continuant_eq_coefficients_prefix_continuant(
            cf, n.suc)
        continued_fraction_prefix_continuant_cons_suc(head, tail, n)
    }
}

/// A displayed nonempty continued fraction has successor prefix numerator
/// equal to the displayed successor prefix numerator.
theorem continued_fraction_prefix_numerator_cons_suc_method(
    cf: ContinuedFraction, head: Nat, tail: List[Nat], n: Nat
) {
    cf.coefficients = List.cons(head, tail)
        implies cf.prefix_numerator(n.suc) =
            continued_fraction_numerator(
                List.cons(head, continued_fraction_prefix_coefficients(tail, n)))
} by {
    if cf.coefficients = List.cons(head, tail) {
        continued_fraction_prefix_numerator_eq_coefficients_prefix_numerator(
            cf, n.suc)
        continued_fraction_prefix_numerator_cons_suc(head, tail, n)
    }
}

/// A displayed nonempty continued fraction has successor prefix denominator
/// equal to the displayed successor prefix denominator.
theorem continued_fraction_prefix_denominator_cons_suc_method(
    cf: ContinuedFraction, head: Nat, tail: List[Nat], n: Nat
) {
    cf.coefficients = List.cons(head, tail)
        implies cf.prefix_denominator(n.suc) =
            continued_fraction_denominator(
                List.cons(head, continued_fraction_prefix_coefficients(tail, n)))
} by {
    if cf.coefficients = List.cons(head, tail) {
        continued_fraction_prefix_denominator_eq_coefficients_prefix_denominator(
            cf, n.suc)
        continued_fraction_prefix_denominator_cons_suc(head, tail, n)
    }
}

/// A displayed nonempty continued fraction has successor prefix convergent
/// equal to the displayed successor prefix convergent.
theorem continued_fraction_prefix_convergent_cons_suc_method(
    cf: ContinuedFraction, head: Nat, tail: List[Nat], n: Nat
) {
    cf.coefficients = List.cons(head, tail)
        implies cf.prefix_convergent(n.suc) =
            continued_fraction_convergent(
                List.cons(head, continued_fraction_prefix_coefficients(tail, n)))
} by {
    if cf.coefficients = List.cons(head, tail) {
        continued_fraction_prefix_convergent_eq_coefficients_prefix_convergent(
            cf, n.suc)
        continued_fraction_prefix_convergent_cons_suc(head, tail, n)
    }
}
