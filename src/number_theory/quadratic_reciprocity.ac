from number_theory.totient import Nat, congr_mod_below_eq, mul_mod_inj_below
from number_theory.modular_inverse import cancel_coprime
from number_theory.congruence import mod_congr_mod_self, congr_mod_symm,
    congr_mod_trans, congr_mod_add, mod_lt
from nat import add_one_right, lt_suc, lte_and_lt, sum_lte, lt_or_lte,
    lt_imp_lte_suc, cross_sum_lte, add_sub, sub_pos, add_cancels_left
numerals Nat

/// Twice the half-size of an odd modulus lies strictly below the modulus.
theorem double_lt_odd_modulus(p: Nat, h: Nat) {
    p = Nat.2 * h + Nat.1 implies p != Nat.0 and Nat.2 * h < p
} by {
    if p = Nat.2 * h + Nat.1 {
        add_one_right(Nat.2 * h)
        (Nat.2 * h).suc = Nat.2 * h + Nat.1
        lt_suc(Nat.2 * h)
        Nat.2 * h < (Nat.2 * h).suc
        Nat.2 * h < p
        Nat.0 <= Nat.2 * h
        lte_and_lt(Nat.0, Nat.2 * h, p)
        Nat.0 < p
        p != Nat.0
        p != Nat.0 and Nat.2 * h < p
    }
}

/// Every element of the positive half of an odd residue system lies strictly
/// below its modulus.
theorem positive_half_below_odd_modulus(p: Nat, h: Nat, x: Nat) {
    p = Nat.2 * h + Nat.1 and x <= h implies p != Nat.0 and x < p
} by {
    if p = Nat.2 * h + Nat.1 and x <= h {
        double_lt_odd_modulus(p, h)
        Nat.2 * h < p
        p != Nat.0
        Nat.0 <= h
        sum_lte(h, Nat.0, h, h)
        h + Nat.0 <= h + h
        h + Nat.0 = h
        h + h = Nat.2 * h
        h <= Nat.2 * h
        lte_and_lt(h, Nat.2 * h, p)
        h < p
        lte_and_lt(x, h, p)
        x < p
        p != Nat.0 and x < p
    }
}

/// The sum of two elements in the positive half of an odd residue system lies
/// strictly below its modulus.
theorem positive_half_sum_below_odd_modulus(
    p: Nat, h: Nat, x: Nat, y: Nat
) {
    p = Nat.2 * h + Nat.1 and x <= h and y <= h implies x + y < p
} by {
    if p = Nat.2 * h + Nat.1 and x <= h and y <= h {
        double_lt_odd_modulus(p, h)
        Nat.2 * h < p
        sum_lte(x, y, h, h)
        x + y <= h + h
        h + h = Nat.2 * h
        x + y <= Nat.2 * h
        lte_and_lt(x + y, Nat.2 * h, p)
        x + y < p
    }
}

/// Every nonzero least residue of an odd modulus has a positive-half
/// representative, either directly or after changing its sign.
theorem nonzero_odd_residue_has_positive_half_representative(
    p: Nat, h: Nat, r: Nat
) {
    p = Nat.2 * h + Nat.1 and Nat.0 < r and r < p implies exists(s: Nat) {
        Nat.0 < s and s <= h and (r = s or r + s = p)
    }
} by {
    if p = Nat.2 * h + Nat.1 and Nat.0 < r and r < p {
        if r <= h {
            Nat.0 < r and r <= h and (r = r or r + r = p)
            exists(s: Nat) {
                Nat.0 < s and s <= h and (r = s or r + s = p)
            }
        } else {
            lt_or_lte(h, r)
            h < r
            let s = p - r
            sub_pos(p, r)
            p - r > Nat.0
            s = p - r
            Nat.0 < s
            r <= p
            add_sub(p, r)
            p - r + r = p
            r + s = p
            lt_imp_lte_suc(h, r)
            h.suc <= r
            add_one_right(h)
            h.suc = h + Nat.1
            h + Nat.1 <= r
            h + Nat.1 + h = Nat.2 * h + Nat.1
            h + Nat.1 + h = p
            cross_sum_lte(h + Nat.1, h, r, s)
            s <= h
            Nat.0 < s and s <= h and (r = s or r + s = p)
            exists(t: Nat) {
                Nat.0 < t and t <= h and (r = t or r + t = p)
            }
        }
    }
}

/// A positive-half element multiplied by a unit has a positive-half
/// representative, either as its least residue or as the complementary one.
theorem unit_multiple_has_positive_half_representative(
    p: Nat, h: Nat, a: Nat, x: Nat
) {
    p = Nat.2 * h + Nat.1 and a.coprime(p) and Nat.0 < x and x <= h
        implies exists(s: Nat) {
            Nat.0 < s and s <= h and
                ((a * x).mod(p) = s or (a * x).mod(p) + s = p)
        }
} by {
    if p = Nat.2 * h + Nat.1 and a.coprime(p) and Nat.0 < x and x <= h {
        positive_half_below_odd_modulus(p, h, x)
        p != Nat.0
        x < p
        mod_lt(a * x, p)
        (a * x).mod(p) < p
        if (a * x).mod(p) = Nat.0 {
            mod_congr_mod_self(a * x, p)
            (a * x).mod(p).congr_mod(a * x, p)
            congr_mod_symm((a * x).mod(p), a * x, p)
            (a * x).congr_mod((a * x).mod(p), p)
            (a * x).congr_mod(Nat.0, p)
            a * Nat.0 = Nat.0
            (a * x).congr_mod(a * Nat.0, p)
            cancel_coprime(a, p, x, Nat.0)
            x.congr_mod(Nat.0, p)
            congr_mod_below_eq(p, x, Nat.0)
            x = Nat.0
            false
        }
        Nat.0 < (a * x).mod(p)
        nonzero_odd_residue_has_positive_half_representative(
            p, h, (a * x).mod(p))
        exists(s: Nat) {
            Nat.0 < s and s <= h and
                ((a * x).mod(p) = s or (a * x).mod(p) + s = p)
        }
    }
}

/// A residue has at most one positive-half representative up to sign for an
/// odd modulus.
theorem positive_half_representative_unique(
    p: Nat, h: Nat, r: Nat, s: Nat, t: Nat
) {
    p = Nat.2 * h + Nat.1
        and Nat.0 < s and s <= h and Nat.0 < t and t <= h
        and (r = s or r + s = p) and (r = t or r + t = p)
        implies s = t
} by {
    if p = Nat.2 * h + Nat.1
        and Nat.0 < s and s <= h and Nat.0 < t and t <= h
        and (r = s or r + s = p) and (r = t or r + t = p) {
        positive_half_sum_below_odd_modulus(p, h, s, t)
        s + t < p
        if s != t {
            if r = s {
                if r = t {
                    s = t
                    false
                } else {
                    r + t = p
                    s + t = p
                    false
                }
            } else {
                r + s = p
                if r = t {
                    t + s = p
                    t + s = s + t
                    s + t = p
                    false
                } else {
                    r + t = p
                    r + s = r + t
                    add_cancels_left(r, s, t)
                    s = t
                    false
                }
            }
        }
        s = t
    }
}

/// Every positive-half element multiplied by a unit has a unique
/// positive-half representative up to sign.
theorem unit_multiple_has_unique_positive_half_representative(
    p: Nat, h: Nat, a: Nat, x: Nat
) {
    p = Nat.2 * h + Nat.1 and a.coprime(p) and Nat.0 < x and x <= h
        implies exists(s: Nat) {
            Nat.0 < s and s <= h and
                ((a * x).mod(p) = s or (a * x).mod(p) + s = p) and
                forall(t: Nat) {
                    (Nat.0 < t and t <= h and
                        ((a * x).mod(p) = t or (a * x).mod(p) + t = p))
                        implies t = s
                }
        }
} by {
    if p = Nat.2 * h + Nat.1 and a.coprime(p) and Nat.0 < x and x <= h {
        unit_multiple_has_positive_half_representative(p, h, a, x)
        let s: Nat satisfy {
            Nat.0 < s and s <= h and
                ((a * x).mod(p) = s or (a * x).mod(p) + s = p)
        }
        forall(t: Nat) {
            if Nat.0 < t and t <= h and
                ((a * x).mod(p) = t or (a * x).mod(p) + t = p) {
                positive_half_representative_unique(
                    p, h, (a * x).mod(p), s, t)
                s = t
                t = s
            }
        }
        exists(u: Nat) {
            Nat.0 < u and u <= h and
                ((a * x).mod(p) = u or (a * x).mod(p) + u = p) and
                forall(t: Nat) {
                    (Nat.0 < t and t <= h and
                        ((a * x).mod(p) = t or (a * x).mod(p) + t = p))
                        implies t = u
                }
        }
    }
}

/// Images of positive-half elements under multiplication by a unit cannot be
/// complementary least residues.
theorem mul_mod_positive_half_not_complementary(
    p: Nat, h: Nat, a: Nat, x: Nat, y: Nat
) {
    p = Nat.2 * h + Nat.1 and a.coprime(p)
        and Nat.0 < x and x <= h and Nat.0 < y and y <= h
        implies (a * x).mod(p) + (a * y).mod(p) != p
} by {
    if p = Nat.2 * h + Nat.1 and a.coprime(p)
        and Nat.0 < x and x <= h and Nat.0 < y and y <= h {
        positive_half_below_odd_modulus(p, h, x)
        p != Nat.0
        if (a * x).mod(p) + (a * y).mod(p) = p {
            mod_congr_mod_self(a * x, p)
            (a * x).mod(p).congr_mod(a * x, p)
            mod_congr_mod_self(a * y, p)
            (a * y).mod(p).congr_mod(a * y, p)
            congr_mod_add(
                (a * x).mod(p), (a * y).mod(p), a * x, a * y, p)
            ((a * x).mod(p) + (a * y).mod(p)).congr_mod(
                a * x + a * y, p)
            congr_mod_symm(
                (a * x).mod(p) + (a * y).mod(p), a * x + a * y, p)
            (a * x + a * y).congr_mod(
                (a * x).mod(p) + (a * y).mod(p), p)
            (a * x + a * y).congr_mod(p, p)
            p.congr_mod(Nat.0, p)
            congr_mod_trans(a * x + a * y, p, Nat.0, p)
            (a * x + a * y).congr_mod(Nat.0, p)
            a * (x + y) = a * x + a * y
            a * Nat.0 = Nat.0
            (a * (x + y)).congr_mod(a * Nat.0, p)
            cancel_coprime(a, p, x + y, Nat.0)
            (x + y).congr_mod(Nat.0, p)
            positive_half_sum_below_odd_modulus(p, h, x, y)
            x + y < p
            congr_mod_below_eq(p, x + y, Nat.0)
            x + y = Nat.0
            false
        }
    }
}

/// Multiplication by a unit is injective on the positive half of an odd
/// residue system, even after each image may be replaced by its negative.
theorem mul_mod_signed_inj_on_positive_half(
    p: Nat, h: Nat, a: Nat, x: Nat, y: Nat
) {
    p = Nat.2 * h + Nat.1 and a.coprime(p)
        and Nat.0 < x and x <= h and Nat.0 < y and y <= h
        and ((a * x).mod(p) = (a * y).mod(p) or
            (a * x).mod(p) + (a * y).mod(p) = p)
        implies x = y
} by {
    if p = Nat.2 * h + Nat.1 and a.coprime(p)
        and Nat.0 < x and x <= h and Nat.0 < y and y <= h
        and ((a * x).mod(p) = (a * y).mod(p) or
            (a * x).mod(p) + (a * y).mod(p) = p) {
        positive_half_below_odd_modulus(p, h, x)
        p != Nat.0
        x < p
        positive_half_below_odd_modulus(p, h, y)
        y < p
        if (a * x).mod(p) = (a * y).mod(p) {
            mul_mod_inj_below(p, a, x, y)
            x = y
        } else {
            (a * x).mod(p) + (a * y).mod(p) = p
            mul_mod_positive_half_not_complementary(p, h, a, x, y)
            false
        }
        x = y
    }
}
