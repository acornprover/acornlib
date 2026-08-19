/// Approximation theory, deepened.
///
/// The classical results that irrational numbers have infinitely many good
/// rational approximations, together with the irrationality measure and the
/// irrationality of the square roots of two and three:
///
///   1. Infinitely many good rational approximations: the continued-fraction
///      limit of a positive-tail coefficient sequence has, beyond every bound,
///      a convergent `p / q` with denominator above the bound and
///      `|alpha - p / q| < 1 / q^2` (Section 1).  Restated from
///      diophantine_approx.ac; for every irrational `alpha` the continued
///      fraction is infinite, so this is the library's form of the classical
///      theorem that every irrational number has infinitely many good
///      rational approximations.
///
///   2. The irrationality measure: if `|alpha - p / q| < 1 / (c * q^2)` holds
///      for infinitely many reduced `p / q` with `1 < c`, then `alpha` is
///      irrational (Section 2).  The proof is the contrapositive: a rational
///      `alpha = a / b` admits no reduced `p / q` with denominator above `b`
///      and `|alpha - p / q| < 1 / (c * q^2)`, because the nonzero integer
///      `a * q - b * p` has absolute value at least one, forcing `c * q < b`
///      and hence `q < b`.
///
///   3. The irrationality of the square roots of two and three (Section 3):
///      the Diophantine equations `x^2 = 2 * y^2` and `x^2 = 3 * y^2` have no
///      positive solution.  The two case restates diophantine.ac; the three
///      case is the classical infinite descent, mirroring the two case.
///      The general statement — `sqrt(n)` is irrational unless `n` is a
///      perfect square — is recorded as a comment below Section 3.
///
///   4. The irrationality of `e` (Section 4) and of `pi` (Section 5) are
///      stated below; neither is proved yet (see the comments there).
from nat import Nat, from_nat, from_nat_zero, from_nat_one, add_comm, add_assoc, add_comm_4,
    mul_comm, mul_assoc, distrib_left, distrib_right, mul_one_left, mul_one_right, add_imp_sub,
    divides_mul, divides_sub, mul_to_zero, mul_cancel_left, gcd_divides_left, gcd_divides_right,
    divides_gcd, gcd_of_prime, two_divides_suc_iff, divides_self, lt_or_lte, lt_add_left,
    lt_add_suc, divisor_lt, lte_mul_both, lte_mul, lte_add_left, lte_add_right, sum_lte,
    only_zero_lte_zero, alt_induction, lt_imp_lte_suc, lt_suc, pos_of_ne_zero, add_sub, divides_lte
from int import Int, abs, abs_from_nat, abs_zero_imp_zero, sub_nat, sub_nat_imp_add, add_from_nat,
    mul_from_nat, abs_neg, neg_neg
from rat import Rat, from_nat_add, from_nat_mul, nat_lt_imp_rat_lt, nat_lte_imp_rat_lte,
    from_nat_nonneg, abs_div, mul_fractions, lt_mul_pos, lte_mul_pos, lt_elim_left_denom,
    zero_lt_imp_pos, pos_ne_zero, abs_non_neg, neg_abs, recip_eq_one_div, mul_inv_cancels_left,
    mul_inv_cancels_right, cross_mul_lt, pos_imp_zero_lt, lt_make_right_denom, sub_self, div_zero,
    cancel_left_num_denom, mul_div_cancels, neg_imp_lt_zero, lt_zero_imp_neg, zero_minus,
    sub_distrib, add_inv_cancels_right, add_inv_cancels_left, neg_sub, lt_cancel_pos_mul_right,
    mul_cancels_div
from real import Real, from_nat_is_from_rat, mul_from_rat, abs_from_rat, from_rat_maintains_lt,
    from_rat_maintains_lte, from_nat_lte_mono, lt_mul_pos_left, lt_mul_pos_right,
    lte_mul_nonneg_left, lte_mul_nonneg_right, gt_zero_imp_pos, lte_self, pos_imp_eq_abs,
    real_lte_imp_rat_lte
from order import lt_imp_lte, lt_of_lte_of_lt, lt_of_lt_of_lte, lt_trans, lte_trans,
    lte_imp_not_lt, lte_ref, lt_and_lte, lt_not_ref, lte_antisymm, lt_lte_trans, lte_lt_trans
from algebra.well_founded import nat_lt_relation, nat_lt_relation_induction_at
from number_theory.coprime import nat_divides_one_imp_one, coprime_mul, coprime_divides_of_divides_mul
from number_theory.goldbach import three_is_prime
from number_theory.diophantine import no_nontrivial_sq_eq_two_sq, nat_sq_double
from number_theory.continued_fraction_convergents import positive_continued_fraction_sequence_tail
from number_theory.continued_fraction_approx import continued_fraction_real_limit,
    continued_fraction_real_convergent_value, rat_sub_fraction, rat_from_nat_positive_ne_zero,
    real_from_rat_sub
from number_theory.diophantine_approx import diophantine_good_approximations_unbounded

numerals Nat

// ============================================================================
// Section 1: infinitely many good rational approximations
// ============================================================================

/// The continued-fraction limit has infinitely many good rational
/// approximations: beyond every bound `n` there is a convergent `p / q` with
/// denominator exceeding `n` and `|alpha - p / q| < 1 / q^2`.
///
/// This restates `diophantine_good_approximations_unbounded` from
/// diophantine_approx.ac; the proof lives there.  For every irrational
/// `alpha` the continued-fraction expansion is infinite (the expansion of a
/// rational number terminates), so this is the library's form of the
/// classical statement that every irrational number has infinitely many good
/// rational approximations.
theorem approximation_deep_infinitely_many_good_approximations(
    coefficients: Nat -> Nat, n: Nat
) {
    positive_continued_fraction_sequence_tail(coefficients) implies
        exists(p: Nat, q: Nat) {
            n < q and
            (continued_fraction_real_limit(coefficients) -
                Real.from_rat(Rat.from_nat(p) / Rat.from_nat(q))).abs <
                Real.from_rat(Rat.1 / Rat.from_nat(q * q))
        }
} by {
    if positive_continued_fraction_sequence_tail(coefficients) {
        diophantine_good_approximations_unbounded(coefficients, n)
        exists(p: Nat, q: Nat) {
            n < q and
            (continued_fraction_real_limit(coefficients) -
                Real.from_rat(Rat.from_nat(p) / Rat.from_nat(q))).abs <
                Real.from_rat(Rat.1 / Rat.from_nat(q * q))
        }
    }
}

// ============================================================================
// Section 2: the irrationality measure
// ============================================================================

/// The rational-to-real embedding reflects strict order.
///
/// This is the strict analogue of `real_lte_imp_rat_lte`; it is proved here
/// because the library's copy (in the private module
/// `real.from_rat_field_hom`) is not importable.
theorem approximation_deep_reflects_lt(p: Rat, q: Rat) {
    Real.from_rat(p) < Real.from_rat(q) implies p < q
} by {
    if Real.from_rat(p) < Real.from_rat(q) {
        lt_imp_lte(Real.from_rat(p), Real.from_rat(q))
        Real.from_rat(p) <= Real.from_rat(q)
        real_lte_imp_rat_lte(p, q)
        p <= q
        if p = q {
            Real.from_rat(p) = Real.from_rat(q)
            lte_self(Real.from_rat(p))
            lte_imp_not_lt(Real.from_rat(p), Real.from_rat(p))
            not (Real.from_rat(p) < Real.from_rat(p))
            false
        }
        p != q
        p < q
    }
}

/// The absolute value of the difference of two embedded naturals is the
/// embedding of the absolute value of their integer difference.
///
/// The integer difference `sub_nat(m, n) = Int.from_nat(m) - Int.from_nat(n)`
/// is the signed difference; taking absolute values on both sides yields the
/// absolute difference of the natural numbers.
theorem approximation_deep_rat_from_nat_sub_abs(m: Nat, n: Nat) {
    (Rat.from_nat(m) - Rat.from_nat(n)).abs = Rat.from_nat(abs(sub_nat(m, n)))
} by {
    if n <= m {
        sub_nat(m, n) = Int.from_nat(m - n)
        add_sub(m, n)
        n <= m
        m - n + n = m
        from_nat_add(m - n, n)
        Rat.from_nat(m - n) + Rat.from_nat(n) = Rat.from_nat((m - n) + n)
        Rat.from_nat(m - n) + Rat.from_nat(n) = Rat.from_nat(m)
        Rat.from_nat(m) - Rat.from_nat(n) = Rat.from_nat(m - n)
        (Rat.from_nat(m) - Rat.from_nat(n)).abs = Rat.from_nat(m - n).abs
        from_nat_nonneg(m - n)
        Rat.0 <= Rat.from_nat(m - n)
        lte_imp_not_lt(Rat.0, Rat.from_nat(m - n))
        not (Rat.from_nat(m - n) < Rat.0)
        if Rat.from_nat(m - n).is_negative {
            neg_imp_lt_zero(Rat.from_nat(m - n))
            Rat.from_nat(m - n) < Rat.0
            false
        }
        not Rat.from_nat(m - n).is_negative
        abs_non_neg(Rat.from_nat(m - n))
        Rat.from_nat(m - n).abs = Rat.from_nat(m - n)
        (Rat.from_nat(m) - Rat.from_nat(n)).abs = Rat.from_nat(m - n)
        abs(sub_nat(m, n)) = abs(Int.from_nat(m - n))
        abs_from_nat(m - n)
        abs(Int.from_nat(m - n)) = m - n
        abs(sub_nat(m, n)) = m - n
        Rat.from_nat(abs(sub_nat(m, n))) = Rat.from_nat(m - n)
        (Rat.from_nat(m) - Rat.from_nat(n)).abs = Rat.from_nat(abs(sub_nat(m, n)))
    }
    if not (n <= m) {
        lt_or_lte(n, m)
        n < m or m <= n
        if n < m {
            lt_imp_lte(n, m)
            n <= m
            not (n <= m)
            false
        }
        if m <= n {
            if m = n {
                lte_ref(n)
                n <= n
                not (n <= m)
                false
            }
            m != n
            lt_or_lte(m, n)
            m < n or n <= m
            if n <= m {
                lte_antisymm(m, n)
                m <= n and n <= m implies m = n
                m = n
                false
            }
            m < n
            lt_imp_lte(m, n)
            m <= n
            add_sub(n, m)
            n - m + m = n
            from_nat_add(n - m, m)
            Rat.from_nat(n - m) + Rat.from_nat(m) = Rat.from_nat((n - m) + m)
            Rat.from_nat(n - m) + Rat.from_nat(m) = Rat.from_nat(n)
            Rat.from_nat(n) - Rat.from_nat(m) = Rat.from_nat(n - m)
            neg_sub(Rat.from_nat(n), Rat.from_nat(m))
            -(Rat.from_nat(n) - Rat.from_nat(m)) = Rat.from_nat(m) - Rat.from_nat(n)
            -(Rat.from_nat(n) - Rat.from_nat(m)) = -(Rat.from_nat(n - m))
            Rat.from_nat(m) - Rat.from_nat(n) = -(Rat.from_nat(n - m))
            (Rat.from_nat(m) - Rat.from_nat(n)).abs = (-(Rat.from_nat(n - m))).abs
            neg_abs(Rat.from_nat(n - m))
            (-(Rat.from_nat(n - m))).abs = Rat.from_nat(n - m).abs
            (Rat.from_nat(m) - Rat.from_nat(n)).abs = Rat.from_nat(n - m).abs
            from_nat_nonneg(n - m)
            Rat.0 <= Rat.from_nat(n - m)
            lte_imp_not_lt(Rat.0, Rat.from_nat(n - m))
            not (Rat.from_nat(n - m) < Rat.0)
            if Rat.from_nat(n - m).is_negative {
                neg_imp_lt_zero(Rat.from_nat(n - m))
                Rat.from_nat(n - m) < Rat.0
                false
            }
            not Rat.from_nat(n - m).is_negative
            abs_non_neg(Rat.from_nat(n - m))
            Rat.from_nat(n - m).abs = Rat.from_nat(n - m)
            (Rat.from_nat(m) - Rat.from_nat(n)).abs = Rat.from_nat(n - m)
            sub_nat(m, n) = -(Int.from_nat(n - m))
            abs(sub_nat(m, n)) = abs(-(Int.from_nat(n - m)))
            abs_neg(Int.from_nat(n - m))
            abs(-(Int.from_nat(n - m))) = abs(Int.from_nat(n - m))
            abs(sub_nat(m, n)) = abs(Int.from_nat(n - m))
            abs_from_nat(n - m)
            abs(Int.from_nat(n - m)) = n - m
            abs(sub_nat(m, n)) = n - m
            Rat.from_nat(abs(sub_nat(m, n))) = Rat.from_nat(n - m)
            (Rat.from_nat(m) - Rat.from_nat(n)).abs = Rat.from_nat(abs(sub_nat(m, n)))
        }
        (Rat.from_nat(m) - Rat.from_nat(n)).abs = Rat.from_nat(abs(sub_nat(m, n)))
    }
}



/// A natural number at least one embeds as a positive rational.
theorem approximation_deep_nat_one_lte_imp_rat_positive(n: Nat) {
    Nat.1 <= n implies Rat.from_nat(n).is_positive
} by {
    if Nat.1 <= n {
        lt_suc(Nat.0)
        Nat.0 < Nat.1
        lt_of_lte_of_lt(Nat.0, Nat.1, n)
        Nat.0 < n
        nat_lt_imp_rat_lt(Nat.0, n)
        Rat.from_nat(Nat.0) < Rat.from_nat(n)
        from_nat_zero[Rat]
        Rat.from_nat(Nat.0) = Rat.0
        Rat.0 < Rat.from_nat(n)
        zero_lt_imp_pos(Rat.from_nat(n))
        Rat.from_nat(n).is_positive
    }
}

/// A rational `alpha = a / b` admits no reduced `p / q` with `q` large and
/// `|alpha - p / q| < 1 / (c * q^2)`: the denominator of any such reduced
/// approximation is bounded by `b`.
///
/// The proof is the standard denominator bound.  Writing
/// `|a / b - p / q| = |a * q - b * p| / (b * q)`, if `a * q != b * p` then the
/// nonzero integer `a * q - b * p` has absolute value at least one, so
/// `1 / (b * q) <= |alpha - p / q| < 1 / (c * q^2)` forces `c * q < b` and
/// hence `q < b`; if `a * q = b * p` then `p / q = a / b`, and since `p` and
/// `q` are coprime, `q` divides `b`, so `q <= b`.  The conclusion is stated
/// as the real inequality `from_nat(q) <= from_nat(b)`, which the main
/// theorem contradicts against the unbounded denominators of the hypothesis.
theorem approximation_deep_rational_denominator_bound(
    a: Nat, b: Nat, p: Nat, q: Nat, c: Nat
) {
    Nat.1 <= b and Nat.1 <= q and Nat.1 < c and q.coprime(p) and
        (Real.from_rat(Rat.from_nat(a) / Rat.from_nat(b)) -
            Real.from_rat(Rat.from_nat(p) / Rat.from_nat(q))).abs <
            Real.from_rat(Rat.1 / Rat.from_nat(c * q * q))
        implies from_nat[Real](q) <= from_nat[Real](b)
} by {
    if Nat.1 <= b and Nat.1 <= q and Nat.1 < c and q.coprime(p) and
            (Real.from_rat(Rat.from_nat(a) / Rat.from_nat(b)) -
                Real.from_rat(Rat.from_nat(p) / Rat.from_nat(q))).abs <
                Real.from_rat(Rat.1 / Rat.from_nat(c * q * q)) {
        // The strict real inequality reflects to the rationals.
        real_from_rat_sub(
            Rat.from_nat(a) / Rat.from_nat(b),
            Rat.from_nat(p) / Rat.from_nat(q))
        Real.from_rat(Rat.from_nat(a) / Rat.from_nat(b)) - Real.from_rat(Rat.from_nat(p) / Rat.from_nat(q)) = Real.from_rat(Rat.from_nat(a) / Rat.from_nat(b) - Rat.from_nat(p) / Rat.from_nat(q))
        abs_from_rat(
            Rat.from_nat(a) / Rat.from_nat(b) -
            Rat.from_nat(p) / Rat.from_nat(q))
        Real.from_rat(Rat.from_nat(a) / Rat.from_nat(b) - Rat.from_nat(p) / Rat.from_nat(q)).abs = Real.from_rat((Rat.from_nat(a) / Rat.from_nat(b) - Rat.from_nat(p) / Rat.from_nat(q)).abs)
        (Real.from_rat(Rat.from_nat(a) / Rat.from_nat(b)) - Real.from_rat(Rat.from_nat(p) / Rat.from_nat(q))).abs = Real.from_rat((Rat.from_nat(a) / Rat.from_nat(b) - Rat.from_nat(p) / Rat.from_nat(q)).abs)
        Real.from_rat((Rat.from_nat(a) / Rat.from_nat(b) - Rat.from_nat(p) / Rat.from_nat(q)).abs) < Real.from_rat(Rat.1 / Rat.from_nat(c * q * q))
        approximation_deep_reflects_lt(
            (Rat.from_nat(a) / Rat.from_nat(b) -
                Rat.from_nat(p) / Rat.from_nat(q)).abs,
            Rat.1 / Rat.from_nat(c * q * q))
        (Rat.from_nat(a) / Rat.from_nat(b) - Rat.from_nat(p) / Rat.from_nat(q)).abs < Rat.1 / Rat.from_nat(c * q * q)
        // The difference of the two fractions is a single fraction with
        // denominator b * q and numerator a * q - b * p.
        lt_imp_lte(Nat.1, b)
        Nat.1 <= b
        lt_of_lte_of_lt(Nat.0, Nat.1, b)
        Nat.0 < b
        rat_from_nat_positive_ne_zero(b)
        Rat.from_nat(b) != Rat.0
        lt_imp_lte(Nat.1, q)
        Nat.1 <= q
        lt_of_lte_of_lt(Nat.0, Nat.1, q)
        Nat.0 < q
        rat_from_nat_positive_ne_zero(q)
        Rat.from_nat(q) != Rat.0
        rat_sub_fraction(
            Rat.from_nat(a), Rat.from_nat(b),
            Rat.from_nat(p), Rat.from_nat(q))
        Rat.from_nat(a) / Rat.from_nat(b) -
                Rat.from_nat(p) / Rat.from_nat(q) =
            (Rat.from_nat(a) * Rat.from_nat(q) -
                Rat.from_nat(p) * Rat.from_nat(b)) /
            (Rat.from_nat(b) * Rat.from_nat(q))
        from_nat_mul(a, q)
        Rat.from_nat(a) * Rat.from_nat(q) = Rat.from_nat(a * q)
        from_nat_mul(p, b)
        Rat.from_nat(p) * Rat.from_nat(b) = Rat.from_nat(p * b)
        from_nat_mul(b, q)
        Rat.from_nat(b) * Rat.from_nat(q) = Rat.from_nat(b * q)
        Rat.from_nat(a) / Rat.from_nat(b) -
                Rat.from_nat(p) / Rat.from_nat(q) =
            (Rat.from_nat(a * q) - Rat.from_nat(p * b)) /
            Rat.from_nat(b * q)
        // The denominator b * q is a positive, nonzero rational.
        lte_mul_both(q, Nat.1, b)
        q * Nat.1 <= q * b
        q * Nat.1 = q
        q <= q * b
        q * b = b * q
        q <= b * q
        lte_trans(Nat.1, q, b * q)
        Nat.1 <= b * q
        approximation_deep_nat_one_lte_imp_rat_positive(b * q)
        Rat.from_nat(b * q).is_positive
        pos_ne_zero(Rat.from_nat(b * q))
        Rat.from_nat(b * q) != Rat.0
        abs_div(
            Rat.from_nat(a * q) - Rat.from_nat(p * b),
            Rat.from_nat(b * q))
        ((Rat.from_nat(a * q) - Rat.from_nat(p * b)) /
            Rat.from_nat(b * q)).abs =
            (Rat.from_nat(a * q) - Rat.from_nat(p * b)).abs /
            Rat.from_nat(b * q).abs
        from_nat_nonneg(b * q)
        Rat.0 <= Rat.from_nat(b * q)
        lte_imp_not_lt(Rat.0, Rat.from_nat(b * q))
        not (Rat.from_nat(b * q) < Rat.0)
        if Rat.from_nat(b * q).is_negative {
            neg_imp_lt_zero(Rat.from_nat(b * q))
            Rat.from_nat(b * q) < Rat.0
            false
        }
        not Rat.from_nat(b * q).is_negative
        abs_non_neg(Rat.from_nat(b * q))
        Rat.from_nat(b * q).abs = Rat.from_nat(b * q)
        ((Rat.from_nat(a * q) - Rat.from_nat(p * b)) /
            Rat.from_nat(b * q)).abs =
            (Rat.from_nat(a * q) - Rat.from_nat(p * b)).abs /
            Rat.from_nat(b * q)
        // The absolute numerator is the embedding of |a * q - b * p|.
        approximation_deep_rat_from_nat_sub_abs(a * q, p * b)
        (Rat.from_nat(a * q) - Rat.from_nat(p * b)).abs =
            Rat.from_nat(abs(sub_nat(a * q, p * b)))
        (Rat.from_nat(a * q) - Rat.from_nat(p * b)).abs / Rat.from_nat(b * q) < Rat.1 / Rat.from_nat(c * q * q)
        Rat.from_nat(abs(sub_nat(a * q, p * b))) / Rat.from_nat(b * q) < Rat.1 / Rat.from_nat(c * q * q)
        // Cross-multiply: |a * q - b * p| * c * q^2 < b * q.
        lt_imp_lte(Nat.1, c)
        Nat.1 <= c
        lt_imp_lte(Nat.1, q)
        lte_mul_both(q, Nat.1, c)
        q * Nat.1 <= q * c
        q * Nat.1 = q
        q <= q * c
        q * c = c * q
        q <= c * q
        lte_mul_both(c * q, Nat.1, q)
        (c * q) * Nat.1 <= (c * q) * q
        (c * q) * Nat.1 = c * q
        c * q <= c * q * q
        lte_trans(Nat.1, q, c * q)
        lte_trans(Nat.1, c * q, c * q * q)
        Nat.1 <= c * q * q
        approximation_deep_nat_one_lte_imp_rat_positive(c * q * q)
        Rat.from_nat(c * q * q).is_positive
        lt_mul_pos(Rat.from_nat(abs(sub_nat(a * q, p * b))) / Rat.from_nat(b * q), Rat.1 / Rat.from_nat(c * q * q), Rat.from_nat(c * q * q))
        (Rat.from_nat(abs(sub_nat(a * q, p * b))) / Rat.from_nat(b * q)) * Rat.from_nat(c * q * q) < (Rat.1 / Rat.from_nat(c * q * q)) * Rat.from_nat(c * q * q)
        mul_cancels_div(Rat.1, Rat.from_nat(c * q * q))
        (Rat.1 / Rat.from_nat(c * q * q)) * Rat.from_nat(c * q * q) = Rat.1
        (Rat.from_nat(abs(sub_nat(a * q, p * b))) / Rat.from_nat(b * q)) * Rat.from_nat(c * q * q) < Rat.1
        (Rat.from_nat(abs(sub_nat(a * q, p * b))) / Rat.from_nat(b * q)) * Rat.from_nat(c * q * q) = (Rat.from_nat(abs(sub_nat(a * q, p * b))) * Rat.from_nat(c * q * q)) / Rat.from_nat(b * q)
        (Rat.from_nat(abs(sub_nat(a * q, p * b))) * Rat.from_nat(c * q * q)) / Rat.from_nat(b * q) < Rat.1
        lt_elim_left_denom(Rat.from_nat(abs(sub_nat(a * q, p * b))) * Rat.from_nat(c * q * q), Rat.1, Rat.from_nat(b * q))
        Rat.from_nat(abs(sub_nat(a * q, p * b))) * Rat.from_nat(c * q * q) < Rat.1 * Rat.from_nat(b * q)
        Rat.1 * Rat.from_nat(b * q) = Rat.from_nat(b * q)
        Rat.from_nat(abs(sub_nat(a * q, p * b))) * Rat.from_nat(c * q * q) < Rat.from_nat(b * q)
        from_nat_mul(abs(sub_nat(a * q, p * b)), c * q * q)
        Rat.from_nat(abs(sub_nat(a * q, p * b)) * (c * q * q)) = Rat.from_nat(abs(sub_nat(a * q, p * b))) * Rat.from_nat(c * q * q)
        Rat.from_nat(abs(sub_nat(a * q, p * b)) * (c * q * q)) < Rat.from_nat(b * q)
        // Case split on whether a * q = b * p.
        if sub_nat(a * q, p * b) = Int.0 {
            // a * q = b * p, and p / q = a / b is reduced, so q divides b.
            Int.from_nat(Nat.0) = Int.0
            sub_nat(a * q, p * b) = Int.from_nat(Nat.0)
            sub_nat_imp_add(a * q, p * b, Nat.0)
            p * b + Nat.0 = a * q
            p * b + Nat.0 = p * b
            p * b = a * q
            a * q = p * b
            divides_self(q)
            q.divides(q)
            divides_mul(q, q, a)
            q.divides(q * a)
            q * a = a * q
            q * a = p * b
            q.divides(p * b)
            q.coprime(p)
            coprime_divides_of_divides_mul(q, p, b)
            q.divides(b)
            divides_lte(q, b)
            b = Nat.0 or q <= b
            if b = Nat.0 {
                Nat.1 <= Nat.0
                lt_suc(Nat.0)
                Nat.0 < Nat.1
                lte_imp_not_lt(Nat.1, Nat.0)
                not (Nat.0 < Nat.1)
                false
            }
            q <= b
            from_nat_lte_mono(q, b)
            from_nat[Real](q) <= from_nat[Real](b)
        }
        if sub_nat(a * q, p * b) != Int.0 {
            // |a * q - b * p| >= 1, so c * q^2 < b * q and q < b.
            if abs(sub_nat(a * q, p * b)) = Nat.0 {
                abs_zero_imp_zero(sub_nat(a * q, p * b))
                sub_nat(a * q, p * b) = Int.0
                false
            }
            abs(sub_nat(a * q, p * b)) != Nat.0
            pos_of_ne_zero(abs(sub_nat(a * q, p * b)))
            Nat.0 < abs(sub_nat(a * q, p * b))
            lt_imp_lte_suc(Nat.0, abs(sub_nat(a * q, p * b)))
            Nat.1 <= abs(sub_nat(a * q, p * b))
            nat_lte_imp_rat_lte(Nat.1, abs(sub_nat(a * q, p * b)))
            Rat.1 <= Rat.from_nat(abs(sub_nat(a * q, p * b)))
            Rat.from_nat(c * q * q).is_positive
            lte_mul_pos(Rat.1, Rat.from_nat(abs(sub_nat(a * q, p * b))), Rat.from_nat(c * q * q))
            Rat.1 * Rat.from_nat(c * q * q) <= Rat.from_nat(abs(sub_nat(a * q, p * b))) * Rat.from_nat(c * q * q)
            Rat.from_nat(abs(sub_nat(a * q, p * b))) * Rat.from_nat(c * q * q) < Rat.from_nat(b * q)
            lte_lt_trans(Rat.1 * Rat.from_nat(c * q * q), Rat.from_nat(abs(sub_nat(a * q, p * b))) * Rat.from_nat(c * q * q), Rat.from_nat(b * q))
            Rat.1 * Rat.from_nat(c * q * q) < Rat.from_nat(b * q)
            Rat.1 * Rat.from_nat(c * q * q) = Rat.from_nat(c * q * q)
            Rat.from_nat(c * q * q) < Rat.from_nat(b * q)
            from_nat_mul(c * q, q)
            Rat.from_nat(c * q) * Rat.from_nat(q) = Rat.from_nat((c * q) * q)
            Rat.from_nat(c * q) * Rat.from_nat(q) = Rat.from_nat(c * q * q)
            from_nat_mul(b, q)
            Rat.from_nat(b) * Rat.from_nat(q) = Rat.from_nat(b * q)
            Rat.from_nat(c * q) * Rat.from_nat(q) < Rat.from_nat(b) * Rat.from_nat(q)
            approximation_deep_nat_one_lte_imp_rat_positive(q)
            Rat.from_nat(q).is_positive
            lt_cancel_pos_mul_right(Rat.from_nat(c * q), Rat.from_nat(b), Rat.from_nat(q))
            Rat.from_nat(c * q) < Rat.from_nat(b)
            from_rat_maintains_lt(Rat.from_nat(c * q), Rat.from_nat(b))
            Real.from_rat(Rat.from_nat(c * q)) < Real.from_rat(Rat.from_nat(b))
            from_nat_is_from_rat(c * q)
            from_nat[Real](c * q) = Real.from_rat(Rat.from_nat(c * q))
            from_nat_is_from_rat(b)
            from_nat[Real](b) = Real.from_rat(Rat.from_nat(b))
            from_nat[Real](c * q) < from_nat[Real](b)
            lte_mul_both(q, Nat.1, c)
            q * Nat.1 <= q * c
            q * Nat.1 = q
            q <= q * c
            q * c = c * q
            q <= c * q
            from_nat_lte_mono(q, c * q)
            from_nat[Real](q) <= from_nat[Real](c * q)
            lte_lt_trans(
                from_nat[Real](q),
                from_nat[Real](c * q),
                from_nat[Real](b))
            from_nat[Real](q) < from_nat[Real](b)
            lt_imp_lte(from_nat[Real](q), from_nat[Real](b))
            from_nat[Real](q) <= from_nat[Real](b)
        }
        from_nat[Real](q) <= from_nat[Real](b)
    }
}

/// The irrationality measure: if `|alpha - p / q| < 1 / (c * q^2)` holds for
/// infinitely many reduced `p / q` with `1 < c`, then `alpha` is irrational.
///
/// The proof is the contrapositive: for a rational `alpha = a / b`, the
/// denominator bound `approximation_deep_rational_denominator_bound` bounds
/// the denominator of every reduced approximation by `b`, so no approximation
/// can have denominator above `b`.
theorem approximation_deep_good_approximations_imp_irrational(
    alpha: Real, c: Nat
) {
    Nat.1 < c implies (
        (forall(bound: Nat) {
            exists(p: Nat, q: Nat) {
                Nat.1 <= q and bound < q and q.coprime(p) and
                (alpha - Real.from_rat(Rat.from_nat(p) / Rat.from_nat(q))).abs <
                    Real.from_rat(Rat.1 / Rat.from_nat(c * q * q))
            }
        }) implies
        not exists(a: Nat, b: Nat) {
            Nat.1 <= b and alpha = Real.from_rat(Rat.from_nat(a) / Rat.from_nat(b))
        }
    )
} by {
    if Nat.1 < c {
        if forall(bound: Nat) {
            exists(p: Nat, q: Nat) {
                Nat.1 <= q and bound < q and q.coprime(p) and
                (alpha - Real.from_rat(Rat.from_nat(p) / Rat.from_nat(q))).abs <
                    Real.from_rat(Rat.1 / Rat.from_nat(c * q * q))
            }
        } {
            if exists(a: Nat, b: Nat) {
                Nat.1 <= b and alpha = Real.from_rat(Rat.from_nat(a) / Rat.from_nat(b))
            } {
                let (a: Nat, b: Nat) satisfy {
                    Nat.1 <= b and alpha = Real.from_rat(Rat.from_nat(a) / Rat.from_nat(b))
                }
                forall(bound: Nat) {
                    exists(p: Nat, q: Nat) {
                        Nat.1 <= q and bound < q and q.coprime(p) and
                        (alpha - Real.from_rat(Rat.from_nat(p) / Rat.from_nat(q))).abs <
                            Real.from_rat(Rat.1 / Rat.from_nat(c * q * q))
                    }
                }
                exists(p: Nat, q: Nat) {
                    Nat.1 <= q and b < q and q.coprime(p) and
                    (alpha - Real.from_rat(Rat.from_nat(p) / Rat.from_nat(q))).abs <
                        Real.from_rat(Rat.1 / Rat.from_nat(c * q * q))
                }
                let (p: Nat, q: Nat) satisfy {
                    Nat.1 <= q and b < q and q.coprime(p) and
                    (alpha - Real.from_rat(Rat.from_nat(p) / Rat.from_nat(q))).abs <
                        Real.from_rat(Rat.1 / Rat.from_nat(c * q * q))
                }
                alpha = Real.from_rat(Rat.from_nat(a) / Rat.from_nat(b))
                (Real.from_rat(Rat.from_nat(a) / Rat.from_nat(b)) - Real.from_rat(Rat.from_nat(p) / Rat.from_nat(q))).abs < Real.from_rat(Rat.1 / Rat.from_nat(c * q * q))
                approximation_deep_rational_denominator_bound(
                    a, b, p, q, c)
                from_nat[Real](q) <= from_nat[Real](b)
                nat_lt_imp_rat_lt(b, q)
                Rat.from_nat(b) < Rat.from_nat(q)
                from_rat_maintains_lt(Rat.from_nat(b), Rat.from_nat(q))
                Real.from_rat(Rat.from_nat(b)) < Real.from_rat(Rat.from_nat(q))
                from_nat_is_from_rat(b)
                from_nat[Real](b) = Real.from_rat(Rat.from_nat(b))
                from_nat_is_from_rat(q)
                from_nat[Real](q) = Real.from_rat(Rat.from_nat(q))
                from_nat[Real](b) < from_nat[Real](q)
                lt_of_lt_of_lte(
                    from_nat[Real](b),
                    from_nat[Real](q),
                    from_nat[Real](b))
                from_nat[Real](b) < from_nat[Real](b)
                lte_self(from_nat[Real](b))
                lte_imp_not_lt(from_nat[Real](b), from_nat[Real](b))
                not (from_nat[Real](b) < from_nat[Real](b))
                false
            }
            not exists(a: Nat, b: Nat) {
                Nat.1 <= b and alpha = Real.from_rat(Rat.from_nat(a) / Rat.from_nat(b))
            }
        }
    }
}

// ============================================================================
// Section 3: the irrationality of √2 and √3
// ============================================================================

/// No positive rational squares to two: √2 is irrational.
///
/// This restates `no_nontrivial_sq_eq_two_sq` from diophantine.ac in the
/// classical form: the Diophantine equation `x^2 = 2 * y^2` has no solution
/// with positive denominator, so a rational `p / q` in lowest terms cannot
/// square to two.
theorem approximation_deep_sqrt_two_irrational(p: Nat, q: Nat) {
    p * p = Nat.2 * (q * q) implies q = Nat.0
} by {
    if p * p = Nat.2 * (q * q) {
        no_nontrivial_sq_eq_two_sq(p, q)
        p = Nat.0 and q = Nat.0
        q = Nat.0
    }
}

/// No positive rational squares to two: √2 is irrational.
///
/// This is the same statement as `approximation_deep_sqrt_two_irrational`
/// with the denominator hypothesis made explicit.
theorem approximation_deep_sqrt_two_no_positive_rational(p: Nat, q: Nat) {
    Nat.0 < q implies p * p != Nat.2 * (q * q)
} by {
    if Nat.0 < q {
        if p * p = Nat.2 * (q * q) {
            no_nontrivial_sq_eq_two_sq(p, q)
            p = Nat.0 and q = Nat.0
            q = Nat.0
            lt_not_ref(Nat.0)
            false
        }
        p * p != Nat.2 * (q * q)
    }
}

/// If three divides a square, it divides the root (three is prime).
theorem three_divides_square_imp_three_divides(x: Nat) {
    Nat.3.divides(x * x) implies Nat.3.divides(x)
} by {
    if Nat.3.divides(x * x) {
        three_is_prime
        gcd_of_prime(Nat.3, x)
        if Nat.3.gcd(x) = Nat.1 {
            Nat.3.coprime(x)
            coprime_divides_of_divides_mul(Nat.3, x, x)
            Nat.3.divides(x)
        } else {
            Nat.3.divides(x)
        }
    }
}

/// The square of a tripled number: (3x)² = 9x².
theorem nat_sq_triple(x: Nat) {
    (Nat.3 * x) * (Nat.3 * x) = Nat.9 * (x * x)
} by {
    (Nat.3 * x) * (Nat.3 * x) = Nat.3 * x * Nat.3 * x
    Nat.3 * x * Nat.3 * x = Nat.3 * Nat.3 * x * x
    Nat.3 * Nat.3 = Nat.9
    Nat.3 * Nat.3 * x * x = Nat.9 * (x * x)
    (Nat.3 * x) * (Nat.3 * x) = Nat.9 * (x * x)
}

/// Infinite descent: a nonzero solution of x² = 3y² yields a smaller one.
/// From x² = 3y² both coordinates are divisible by three, x = 3a and
/// y = 3b, and then a² = 3b² with a < x.
theorem sq_eq_three_sq_descent(x: Nat, y: Nat) {
    x != Nat.0 and x * x = Nat.3 * (y * y) implies
        exists(a: Nat, b: Nat) {
            a < x and Nat.3 * a = x and a * a = Nat.3 * (b * b)
        }
} by {
    if x != Nat.0 and x * x = Nat.3 * (y * y) {
        // 3 | x·x, hence 3 | x, x = 3a.
        Nat.3.divides(Nat.3 * (y * y))
        Nat.3 * (y * y) = x * x
        Nat.3.divides(x * x)
        three_divides_square_imp_three_divides(x)
        Nat.3.divides(x)
        let (a: Nat) satisfy { Nat.3 * a = x }
        // a is a proper smaller root: a < x.
        mul_to_zero(Nat.3, a)
        if Nat.3 * a != Nat.0 {
            a != Nat.0
        }
        a != Nat.0
        a * Nat.3 = Nat.3 * a
        a * Nat.3 = x
        Nat.1 < Nat.3
        divisor_lt(a, Nat.3, x)
        a < x
        // y·y = 3·a·a from (3a)² = 3y².
        Nat.3 * a = x
        x * x = (Nat.3 * a) * (Nat.3 * a)
        x * x = Nat.3 * (y * y)
        (Nat.3 * a) * (Nat.3 * a) = Nat.3 * (y * y)
        nat_sq_triple(a)
        (Nat.3 * a) * (Nat.3 * a) = Nat.9 * (a * a)
        Nat.9 * (a * a) = Nat.3 * (y * y)
        Nat.3 * (Nat.3 * (a * a)) = Nat.3 * (y * y)
        mul_cancel_left(Nat.3, Nat.3 * (a * a), y * y)
        Nat.3 * (a * a) = y * y
        // 3 | y·y, hence 3 | y, y = 3b.
        Nat.3.divides(Nat.3 * (a * a))
        Nat.3 * (a * a) = y * y
        Nat.3.divides(y * y)
        three_divides_square_imp_three_divides(y)
        Nat.3.divides(y)
        let (b: Nat) satisfy { Nat.3 * b = y }
        // a·a = 3·b·b from y = 3b.
        Nat.3 * b = y
        y * y = (Nat.3 * b) * (Nat.3 * b)
        Nat.3 * (a * a) = y * y
        Nat.3 * (a * a) = (Nat.3 * b) * (Nat.3 * b)
        nat_sq_triple(b)
        (Nat.3 * b) * (Nat.3 * b) = Nat.9 * (b * b)
        Nat.3 * (a * a) = Nat.9 * (b * b)
        Nat.3 * (a * a) = Nat.3 * (Nat.3 * (b * b))
        mul_cancel_left(Nat.3, a * a, Nat.3 * (b * b))
        a * a = Nat.3 * (b * b)
        // assemble the witness.
        a < x and Nat.3 * a = x and a * a = Nat.3 * (b * b)
        exists(a2: Nat, b2: Nat) {
            a2 < x and Nat.3 * a2 = x and a2 * a2 = Nat.3 * (b2 * b2)
        }
    }
}

/// The infinite descent of `sq_eq_three_sq_descent`, run by well-founded
/// induction on the first coordinate: x² = 3y² forces x = 0.
theorem sq_eq_three_sq_zero(x: Nat) {
    forall(y: Nat) { x * x = Nat.3 * (y * y) implies x = Nat.0 }
} by {
    let f: Nat -> Bool = function(t: Nat) {
        forall(y: Nat) { t * t = Nat.3 * (y * y) implies t = Nat.0 }
    }
    forall(z: Nat) {
        if forall(k: Nat) { nat_lt_relation(k, z) implies f(k) } {
            forall(y: Nat) {
                if z * z = Nat.3 * (y * y) {
                    if z = Nat.0 {
                        z = Nat.0
                    } else {
                        z != Nat.0 and z * z = Nat.3 * (y * y)
                        sq_eq_three_sq_descent(z, y)
                        let (a: Nat, b: Nat) satisfy {
                            a < z and Nat.3 * a = z and a * a = Nat.3 * (b * b)
                        }
                        nat_lt_relation(a, z) = (a < z)
                        nat_lt_relation(a, z)
                        forall(k: Nat) { nat_lt_relation(k, z) implies f(k) }
                        nat_lt_relation(a, z) implies f(a)
                        f(a)
                        f(a) =
                            forall(w: Nat) { a * a = Nat.3 * (w * w) implies a = Nat.0 }
                        forall(w: Nat) { a * a = Nat.3 * (w * w) implies a = Nat.0 }
                        a * a = Nat.3 * (b * b) implies a = Nat.0
                        a = Nat.0
                        Nat.3 * a = z
                        Nat.3 * Nat.0 = Nat.0
                        z = Nat.0
                        false
                    }
                }
            }
            forall(y: Nat) { z * z = Nat.3 * (y * y) implies z = Nat.0 }
            f(z) = forall(w: Nat) { z * z = Nat.3 * (w * w) implies z = Nat.0 }
            f(z)
        }
    }
    nat_lt_relation_induction_at(f, x)
    f(x)
    f(x) = forall(y: Nat) { x * x = Nat.3 * (y * y) implies x = Nat.0 }
    forall(y: Nat) { x * x = Nat.3 * (y * y) implies x = Nat.0 }
}

/// The Diophantine equation x² = 3y² has no nonzero natural solution.
theorem no_nontrivial_sq_eq_three_sq(x: Nat, y: Nat) {
    x * x = Nat.3 * (y * y) implies x = Nat.0 and y = Nat.0
} by {
    if x * x = Nat.3 * (y * y) {
        sq_eq_three_sq_zero(x)
        forall(w: Nat) { x * x = Nat.3 * (w * w) implies x = Nat.0 }
        x * x = Nat.3 * (y * y) implies x = Nat.0
        x = Nat.0
        x * x = Nat.3 * (y * y)
        Nat.0 * Nat.0 = Nat.0
        Nat.0 = Nat.3 * (y * y)
        Nat.3 * (y * y) = Nat.0
        mul_to_zero(Nat.3, y * y)
        if Nat.3 = Nat.0 {
            false
        }
        y * y = Nat.0
        mul_to_zero(y, y)
        if y = Nat.0 {
        } else {
            false
        }
        y = Nat.0
        x = Nat.0 and y = Nat.0
    }
}

/// No positive rational squares to three: √3 is irrational.
///
/// The Diophantine equation `x^2 = 3 * y^2` has no solution with positive
/// denominator, so a rational `p / q` in lowest terms cannot square to three.
theorem approximation_deep_sqrt_three_irrational(p: Nat, q: Nat) {
    p * p = Nat.3 * (q * q) implies q = Nat.0
} by {
    if p * p = Nat.3 * (q * q) {
        no_nontrivial_sq_eq_three_sq(p, q)
        p = Nat.0 and q = Nat.0
        q = Nat.0
    }
}

/// No positive rational squares to three: √3 is irrational.
theorem approximation_deep_sqrt_three_no_positive_rational(p: Nat, q: Nat) {
    Nat.0 < q implies p * p != Nat.3 * (q * q)
} by {
    if Nat.0 < q {
        if p * p = Nat.3 * (q * q) {
            no_nontrivial_sq_eq_three_sq(p, q)
            p = Nat.0 and q = Nat.0
            q = Nat.0
            lt_not_ref(Nat.0)
            false
        }
        p * p != Nat.3 * (q * q)
    }
}

// The general statement — √N is irrational unless N is a perfect square —
// requires the prime-factorization form of "not a perfect square".  In the
// Diophantine encoding x² = N·y² it reads:
//
// theorem approximation_deep_sqrt_n_irrational_unless_square(n: Nat) {
//     not exists(m: Nat) { n = m * m } implies forall(p: Nat, q: Nat) {
//         p * p = n * (q * q) implies q = Nat.0
//     }
// }
//
// For n = 2 and n = 3 the statement is proved above (the descent argument
// handles the parity of the exponent of two, respectively of three, in the
// prime factorization, exactly as in diophantine.ac).  The general case
// would proceed by descent on the exponent of a prime dividing n with odd
// exponent; the library does not yet assemble the prime-factorization form
// of "not a perfect square" together with the descent, so the statement is
// recorded here without proof.

// ============================================================================
// Section 4: the irrationality of e
// ============================================================================

// The irrationality of e.  The library defines the real exponential function
// exp(x) as the limit of the partial sums of exp_term(x, n) = x^n / n!
// (real/exp.ac), so Euler's number e = exp(Real.1) exists as a real; but the
// library does not yet prove it irrational.
//
// The classical proof (Fourier) runs as follows.  From the series
// e = sum_{k >= 0} 1 / k!, multiply by N!:
//
//     N! * e = sum_{k = 0}^{N} N! / k! + sum_{k > N} N! / k!.
//
// The first sum is an integer (N! / k! = (k + 1) ... N is a natural), while
// the tail satisfies 0 < sum_{k > N} N! / k! < 1: each term equals
// 1 / ((N + 1) ... k) <= 1 / (N + 1)^(k - N), and the geometric series
// bounds the tail by 1 / N.  Hence N! * e is strictly between the integers
// sum_{k = 0}^{N} N! / k! and that integer plus one, so it is not an
// integer; since N was arbitrary and a rational a / b would make N! * e an
// integer for every N >= b (because b divides N!), e cannot be rational.
//
// The library lacks three ingredients for this proof: the divisibility
// N! / k! for k <= N (a factorial-divisibility lemma), the geometric tail
// bound on the exp series (real/exp.ac bounds the tail of exp_term in other
// forms, but not this one), and a "strictly between consecutive integers is
// not an integer" lemma for reals (there is no integer-ness predicate on
// Real).  The statement is recorded here for future work.
//
// theorem approximation_deep_e_irrational {
//     not exists(a: Nat, b: Nat) {
//         Nat.1 <= b and exp(Real.1) = Real.from_rat(Rat.from_nat(a) / Rat.from_nat(b))
//     }
// }

// ============================================================================
// Section 5: the irrationality of π
// ============================================================================

// The irrationality of π.  The library defines π = 2 * π/2 in real/pi.ac
// (the infimum of the first zeros of Real.cos on the positive axis) and proves
// π > 0, π < 4, Real.sin π = 0, Real.cos π = -1; but the irrationality of π is not yet
// formalized.
//
// The classical proofs are deep.  Niven's proof (1947) considers the
// integral of x^n (π - x)^n Real.sin x over [0, π] with x = π a / b: for the
// integrand polynomial P(x) = x^n (a - b x)^n / n!, repeated integration by
// parts shows the integral is a rational with denominator dividing b^n (and
// in fact an integer multiple of 1 / b^n), while the crude bound
// 0 < integral < π^(n + 1) a^n / n! tends to zero as n grows, contradicting
// the lower bound 1 / b^n for any rational value.  Lambert's proof instead
// derives the continued fraction tan x = x / (1 - x² / (3 - x² / (5 - ...)))
// and shows it is irrational at x = π / 4 (since tan(π / 4) = 1).
//
// The library's calculus machinery (real/pi.ac, real/trig.ac,
// real/integral_exp.ac, real/derivative_*.ac) provides Real.sin, Real.cos, the
// derivative API, and integration, so Niven's proof is in principle
// formalizable, but it requires the polynomial-integration machinery for
// x^n (π - x)^n Real.sin x and the positivity/bound estimates; it is recorded
// here as future work.
//
// theorem approximation_deep_pi_irrational {
//     not exists(a: Nat, b: Nat) {
//         Nat.1 <= b and pi = Real.from_rat(Rat.from_nat(a) / Rat.from_nat(b))
//     }
// }
