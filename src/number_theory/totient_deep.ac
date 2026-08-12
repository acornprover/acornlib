/// Euler's totient, deepened: the product formula in the library's natural
/// form, multiplicativity, the classical parity result, and the half-modulus
/// characterisation.
///
/// The classical product formula `phi(n) = n * prod_{p | n} (1 - 1/p)` is
/// stated here in the natural-valued form available in this library: the local
/// factor attached to a prime power `p^k || n` is `p^(k-1) * (p - 1)` (the
/// formula's `p^k * (1 - 1/p)`), and `phi` is multiplicative on coprime
/// arguments, so the local factors multiply to `phi(n)`.  The general statement
/// over the whole prime factorisation is recorded in a comment below; the
/// prime-power and two-prime cases are proved here, together with
/// `phi(p^k) = p^k - p^(k-1)`, the multiplicativity law, the classical parity
/// result `phi(n)` even for `n > 2` (by pairing each reduced residue `a` with
/// its reflection `n - a`), and the forward half of the characterisation
/// `phi(n) = n / 2` iff `n = 2^k`.
from nat import Nat, add_comm, add_assoc, add_cancels_right, add_imp_sub,
    add_imp_sub_left,
    lt_or_lte, lt_trans, lt_and_lte,
    lte_add_left, lt_add_left,
    mul_comm, mul_two_left,
    exp_add, exp_one, exp_zero, zero_or_suc, pos_of_ne_zero,
    divides_sub, divides_gcd, gcd_divides_left, gcd_divides_right
from list import List, map, product
from list import map_length, filter_preserves_unique,
    filter_equivalent_to_and, filter_contained_by_and,
    map_contains, map_contains_of_contains,
    is_permutation, permutation_preserves_length, unique_same_contains_imp_permutation
from data.finite.finite_fiber_partition import locally_injective_map_is_unique
from number_theory.coprime import coprime_zero_left_imp_one, nat_divides_one_imp_one
from number_theory.factorisation import count_prime_factor
from number_theory.totient import nat_totient, count_coprime_to, count_coprime_to_suc_yes,
    count_coprime_to_suc_no, coprime_residues, coprime_residues_below,
    coprime_residues_below_suc_yes, coprime_residues_below_suc_no,
    coprime_residues_contains_imp, coprime_residues_contains_intro,
    coprime_residues_unique, totient_pq, totient_p_pow_factored, totient_mul_coprime_positive,
    nat_totient_mul_coprime
from number_theory.totient_sums import totient_prime_power, lte_neq_imp_lt
from number_theory.carmichael import two_is_prime
numerals Nat

/// A natural number is even: twice some natural.
define is_even(n: Nat) -> Bool {
    exists(k: Nat) { n = Nat.2 * k }
}

/// The reflection of a residue modulo `n`: `a -> n - a`.  Pairs each reduced
/// residue with another one; on the reduced residues below `n` it is an
/// involution, which makes the parity of the totient computable by pairing.
define reflect_residue(n: Nat, a: Nat) -> Nat {
    n - a
}

/// Reflecting twice returns the original residue: natural subtraction undoes
/// itself as long as nothing was clipped at zero.
theorem reflect_residue_involution(n: Nat, a: Nat) {
    a <= n implies reflect_residue(n, reflect_residue(n, a)) = a
} by {
    if a <= n {
        let (c: Nat) satisfy {
            a + c = n
        }
        add_imp_sub_left(a, c, n)
        n - a = c
        reflect_residue(n, a) = c
        add_imp_sub(a, c, n)
        n - c = a
        reflect_residue(n, c) = a
        reflect_residue(n, reflect_residue(n, a)) = a
    }
}

/// A common divisor of the modulus and a residue divides the reflection.
theorem divides_reflect_residue(n: Nat, a: Nat, d: Nat) {
    d.divides(n) and d.divides(a) implies d.divides(reflect_residue(n, a))
} by {
    if d.divides(n) and d.divides(a) {
        divides_sub(n, a, d)
        d.divides(n - a)
        d.divides(reflect_residue(n, a))
    }
}

/// A common divisor of the modulus and the reflection divides the residue.
theorem divides_of_divides_reflect(n: Nat, a: Nat, d: Nat) {
    a <= n and d.divides(n) and d.divides(reflect_residue(n, a)) implies d.divides(a)
} by {
    if a <= n and d.divides(n) and d.divides(reflect_residue(n, a)) {
        divides_reflect_residue(n, reflect_residue(n, a), d)
        d.divides(reflect_residue(n, reflect_residue(n, a)))
        reflect_residue_involution(n, a)
        reflect_residue(n, reflect_residue(n, a)) = a
        d.divides(a)
    }
}

/// The reflection of a reduced residue is a reduced residue: a common factor
/// of the reflection and the modulus would be a common factor of the residue
/// and the modulus, of which there is none but one.
theorem reflect_residue_coprime(n: Nat, a: Nat) {
    a <= n and a.coprime(n) implies reflect_residue(n, a).coprime(n)
} by {
    if a <= n and a.coprime(n) {
        gcd_divides_left(reflect_residue(n, a), n)
        reflect_residue(n, a).gcd(n).divides(reflect_residue(n, a))
        gcd_divides_right(reflect_residue(n, a), n)
        reflect_residue(n, a).gcd(n).divides(n)
        divides_of_divides_reflect(n, a, reflect_residue(n, a).gcd(n))
        reflect_residue(n, a).gcd(n).divides(a)
        divides_gcd(reflect_residue(n, a).gcd(n), a, n)
        reflect_residue(n, a).gcd(n).divides(a.gcd(n))
        a.coprime(n) = (a.gcd(n) = Nat.1)
        a.gcd(n) = Nat.1
        reflect_residue(n, a).gcd(n).divides(Nat.1)
        nat_divides_one_imp_one(reflect_residue(n, a).gcd(n))
        reflect_residue(n, a).gcd(n) = Nat.1
        reflect_residue(n, a).coprime(n) = (reflect_residue(n, a).gcd(n) = Nat.1)
        reflect_residue(n, a).coprime(n)
    }
}

/// The reflection of a positive residue below the modulus is again one.
theorem reflect_residue_in_range(n: Nat, a: Nat) {
    Nat.0 < a and a < n implies Nat.0 < reflect_residue(n, a) and reflect_residue(n, a) < n
} by {
    if Nat.0 < a and a < n {
        a <= n
        let (c: Nat) satisfy {
            a + c = n
        }
        add_imp_sub_left(a, c, n)
        n - a = c
        reflect_residue(n, a) = c
        if c = Nat.0 {
            a + Nat.0 = n
            a = n
            a < a
            false
        }
        c != Nat.0
        Nat.0 < c
        Nat.0 < reflect_residue(n, a)
        c <= a + c
        c <= n
        if c = n {
            a + n = n
            Nat.0 + n = n
            a + n = Nat.0 + n
            add_cancels_right(n, a, Nat.0)
            a = Nat.0
            Nat.0 < Nat.0
            false
        }
        c != n
        c < n
        reflect_residue(n, a) < n
        (Nat.0 < reflect_residue(n, a) and reflect_residue(n, a) < n)
    }
}

/// A residue and its reflection add to the modulus: the pairing identity
/// behind the evenness argument.
theorem reflect_residue_adds_to_modulus(n: Nat, a: Nat) {
    a <= n implies a + reflect_residue(n, a) = n
} by {
    if a <= n {
        let (c: Nat) satisfy {
            a + c = n
        }
        add_imp_sub_left(a, c, n)
        n - a = c
        reflect_residue(n, a) = c
        a + reflect_residue(n, a) = n
    }
}

/// The Euler-product factor of `n` at a prime `p`: `p^(a-1) * (p - 1)` where
/// `a` is the multiplicity of `p` in the prime factorisation of `n`.  In the
/// classical notation this is the factor `p^a * (1 - 1/p)` of the product
/// formula, written without fractions.
define totient_product_factor(n: Nat, p: Nat) -> Nat {
    p.pow(count_prime_factor(p, n) - Nat.1) * (p - Nat.1)
}

// The general product formula, in the library's natural form:
//
//   phi(n) = prod_{p | n} (p^(a_p - 1) * (p - 1)),   a_p = count_prime_factor(p, n)
//
// with the product over the distinct prime divisors of `n` (the deduplicated
// factorisation list).  This is `n * prod_{p | n} (1 - 1/p)` written without
// fractions.  A proof needs the full multiplicativity of `phi` over the prime
// factorisation (an induction over the factor list using
// `totient_mul_coprime_positive` at each step, with coprimality of a prime
// against the product of the remaining factors, via
// `coprime_imp_no_shared_prime_factor`), plus the local prime-power form
// `totient_prime_power_product_form` below.  The prime-power and two-prime
// cases are proved here; the general statement is left for the factorisation
// lane.
//
// theorem totient_product_formula(n: Nat) {
//     Nat.1 <= n implies
//         n.totient = product[Nat](map(prime_factorisation(n).unique, totient_product_factor(n)))
// }

/// Euler's totient at a prime power: `totient(p^k) = p^k - p^(k-1)` for a
/// prime `p` and `k >= 1`.  Restatement of `totient_prime_power` from
/// `number_theory.totient_sums`.
theorem totient_prime_power_deep(p: Nat, k: Nat) {
    Nat.1 <= k and p.is_prime implies (p.pow(k)).totient = p.pow(k) - p.pow(k - Nat.1)
} by {
    if Nat.1 <= k and p.is_prime {
        totient_prime_power(p, k)
        (p.pow(k)).totient = p.pow(k) - p.pow(k - Nat.1)
    }
}

/// Product-form of the prime-power totient:
/// `totient(p^k) = p^(k-1) * (p - 1)` for a prime `p` and `k >= 1`.  This is
/// the product formula's local factor `p^k * (1 - 1/p)` at `n = p^k`, written
/// in naturals.
theorem totient_prime_power_product_form(p: Nat, k: Nat) {
    Nat.1 <= k and p.is_prime implies (p.pow(k)).totient = p.pow(k - Nat.1) * (p - Nat.1)
} by {
    if Nat.1 <= k and p.is_prime {
        zero_or_suc(k)
        if k = Nat.0 {
            not (Nat.1 <= Nat.0)
            Nat.1 <= Nat.0
            false
        }
        let q: Nat satisfy { k = q.suc }
        k = q.suc
        totient_p_pow_factored(p, q)
        (p.pow(q.suc)).totient = (p - Nat.1) * p.pow(q)
        p.pow(q.suc) = p.pow(k)
        (p.pow(k)).totient = (p - Nat.1) * p.pow(q)
        add_imp_sub(q, Nat.1, k)
        k - Nat.1 = q
        p.pow(q) = p.pow(k - Nat.1)
        (p.pow(k)).totient = (p - Nat.1) * p.pow(k - Nat.1)
        (p - Nat.1) * p.pow(k - Nat.1) = p.pow(k - Nat.1) * (p - Nat.1)
        (p.pow(k)).totient = p.pow(k - Nat.1) * (p - Nat.1)
    }
}


/// Euler's totient is multiplicative on coprime arguments:
/// `nat_totient(m * n) = nat_totient(m) * nat_totient(n)` for `m.coprime(n)`.
/// Restatement of `nat_totient_mul_coprime` from `number_theory.totient`.
theorem totient_mul_coprime_deep(m: Nat, n: Nat) {
    m.coprime(n) implies nat_totient(m * n) = nat_totient(m) * nat_totient(n)
} by {
    if m.coprime(n) {
        nat_totient_mul_coprime(m, n)
        nat_totient(m * n) = nat_totient(m) * nat_totient(n)
    }
}


/// From `not (a < b)` conclude `b <= a`.
theorem not_lt_imp_lte(a: Nat, b: Nat) {
    not (a < b) implies b <= a
} by {
    if not (a < b) {
        lt_or_lte(a, b)
        if a < b {
            false
        }
        b <= a
    }
}

/// Cancelling a common addend on the left in a strict inequality.
theorem add_lt_cancel_left(a: Nat, b: Nat, c: Nat) {
    a + b < a + c implies b < c
} by {
    if a + b < a + c {
        if not (b < c) {
            not_lt_imp_lte(b, c)
            c <= b
            lte_add_left(a, c, b)
            a + c <= a + b
            lt_and_lte(a + b, a + c, a + b)
            a + b < a + b
            false
        }
        b < c
    }
}

/// A divisor of `n` that is coprime to `n` is one.
theorem coprime_divisor_one(n: Nat, d: Nat) {
    d.divides(n) and d.coprime(n) implies d = Nat.1
} by {
    if d.divides(n) and d.coprime(n) {
        d.coprime(n) = (d.gcd(n) = Nat.1)
        d.gcd(n) = Nat.1
        divides_gcd(d, d, n)
        d.divides(d.gcd(n))
        d.divides(Nat.1)
        nat_divides_one_imp_one(d)
        d = Nat.1
    }
}

/// True when `a` lies in the lower half of the range `[0, n)`: `a + a < n`.
define residue_lower_half(n: Nat, a: Nat) -> Bool {
    a + a < n
}

/// True when `a` lies in the upper half of the range `[0, n)`: `n < a + a`.
define residue_upper_half(n: Nat, a: Nat) -> Bool {
    n < a + a
}

/// The reduced residues below `n` that lie in the lower half of `[0, n)`.
define lower_half_residues(n: Nat) -> List[Nat] {
    coprime_residues(n).filter(residue_lower_half(n))
}

/// The reduced residues below `n` that lie in the upper half of `[0, n)`.
define upper_half_residues(n: Nat) -> List[Nat] {
    coprime_residues(n).filter(residue_upper_half(n))
}

/// A reduced residue below `n > 2` is nonzero: zero is coprime only to one.
theorem residue_nonzero(n: Nat, x: Nat) {
    Nat.2 < n and coprime_residues(n).contains(x) implies x != Nat.0
} by {
    if Nat.2 < n and coprime_residues(n).contains(x) {
        if x = Nat.0 {
            coprime_residues_contains_imp(n, x)
            x < n and x.coprime(n)
            x.coprime(n)
            Nat.0.coprime(n)
            coprime_zero_left_imp_one(n)
            n = Nat.1
            Nat.2 < Nat.1
            false
        }
        x != Nat.0
    }
}

/// No reduced residue below `n > 2` is exactly half the modulus: `x + x = n`
/// would make `x` a coprime divisor of `n`, hence `n = 2`.
theorem residue_not_half(n: Nat, x: Nat) {
    Nat.2 < n and coprime_residues(n).contains(x) implies x + x != n
} by {
    if Nat.2 < n and coprime_residues(n).contains(x) {
        if x + x = n {
            mul_two_left(x)
            Nat.2 * x = x + x
            Nat.2 * x = n
            mul_comm(Nat.2, x)
            Nat.2 * x = x * Nat.2
            x * Nat.2 = n
            x.divides(n) = exists(c: Nat) { x * c = n }
            x.divides(n)
            coprime_residues_contains_imp(n, x)
            x < n and x.coprime(n)
            x.coprime(n)
            coprime_divisor_one(n, x)
            x = Nat.1
            x + x = Nat.1 + Nat.1
            Nat.1 + Nat.1 = Nat.2
            n = Nat.2
            Nat.2 < Nat.2
            false
        }
        x + x != n
    }
}

/// Every reduced residue below `n > 2` lies strictly in one of the two halves.
theorem residue_lower_or_upper(n: Nat, x: Nat) {
    Nat.2 < n and coprime_residues(n).contains(x)
        implies residue_lower_half(n, x) or residue_upper_half(n, x)
} by {
    if Nat.2 < n and coprime_residues(n).contains(x) {
        residue_lower_half(n, x) = (x + x < n)
        residue_upper_half(n, x) = (n < x + x)
        lt_or_lte(x + x, n)
        if x + x < n {
            residue_lower_half(n, x)
        }
        if not (x + x < n) {
            residue_not_half(n, x)
            x + x != n
            not_lt_imp_lte(x + x, n)
            n <= x + x
            lte_neq_imp_lt(n, x + x)
            n < x + x
            residue_upper_half(n, x)
        }
        residue_lower_half(n, x) or residue_upper_half(n, x)
    }
}

/// Not lower half implies upper half, for a reduced residue below `n > 2`.
theorem residue_not_lower_imp_upper(n: Nat, x: Nat) {
    Nat.2 < n and coprime_residues(n).contains(x) and not residue_lower_half(n, x)
        implies residue_upper_half(n, x)
} by {
    if Nat.2 < n and coprime_residues(n).contains(x) and not residue_lower_half(n, x) {
        residue_lower_half(n, x) = (x + x < n)
        not (x + x < n)
        residue_not_half(n, x)
        x + x != n
        not_lt_imp_lte(x + x, n)
        n <= x + x
        if n = x + x {
            x + x = n
            false
        }
        n != x + x
        lte_neq_imp_lt(n, x + x)
        n < x + x
        residue_upper_half(n, x) = (n < x + x)
        residue_upper_half(n, x)
    }
}

/// Upper half implies not lower half, for a reduced residue below `n > 2`.
theorem residue_upper_imp_not_lower(n: Nat, x: Nat) {
    Nat.2 < n and coprime_residues(n).contains(x) and residue_upper_half(n, x)
        implies not residue_lower_half(n, x)
} by {
    if Nat.2 < n and coprime_residues(n).contains(x) and residue_upper_half(n, x) {
        residue_upper_half(n, x) = (n < x + x)
        n < x + x
        if x + x < n {
            lt_trans(x + x, n, x + x)
            x + x < x + x
            false
        }
        not (x + x < n)
        residue_lower_half(n, x) = (x + x < n)
        not residue_lower_half(n, x)
    }
}

/// A coprime residue cannot be exactly half the modulus when `n > 2`: `2j = n`
/// would make `j` a coprime divisor of `n`, hence `n = 2`.
theorem coprime_not_half(n: Nat, j: Nat) {
    Nat.2 < n and j.coprime(n) implies j + j != n
} by {
    if Nat.2 < n and j.coprime(n) {
        if j + j = n {
            mul_two_left(j)
            Nat.2 * j = j + j
            Nat.2 * j = n
            mul_comm(Nat.2, j)
            Nat.2 * j = j * Nat.2
            j * Nat.2 = n
            j.divides(n) = exists(c: Nat) { j * c = n }
            j.divides(n)
            coprime_divisor_one(n, j)
            j = Nat.1
            j + j = Nat.1 + Nat.1
            Nat.1 + Nat.1 = Nat.2
            n = Nat.2
            Nat.2 < Nat.2
            false
        }
        j + j != n
    }
}

/// The number of lower-half reduced residues in `[0, k)`: `j` counts exactly
/// when it is coprime to `n` and lies in the lower half.
define count_lower_half(n: Nat, k: Nat) -> Nat {
    match k {
        Nat.zero {
            Nat.0
        }
        Nat.suc(j) {
            if j.coprime(n) and residue_lower_half(n, j) {
                count_lower_half(n, j) + Nat.1
            } else {
                count_lower_half(n, j)
            }
        }
    }
}

/// The number of upper-half reduced residues in `[0, k)`: `j` counts exactly
/// when it is coprime to `n` and lies in the upper half.
define count_upper_half(n: Nat, k: Nat) -> Nat {
    match k {
        Nat.zero {
            Nat.0
        }
        Nat.suc(j) {
            if j.coprime(n) and residue_upper_half(n, j) {
                count_upper_half(n, j) + Nat.1
            } else {
                count_upper_half(n, j)
            }
        }
    }
}

/// Recurrence: a coprime lower-half index adds one to the lower-half count.
theorem count_lower_half_suc_yes(n: Nat, j: Nat) {
    j.coprime(n) and residue_lower_half(n, j)
        implies count_lower_half(n, j.suc) = count_lower_half(n, j) + Nat.1
}

/// Recurrence: an index that is not a coprime lower-half element leaves the
/// lower-half count alone.
theorem count_lower_half_suc_no(n: Nat, j: Nat) {
    not (j.coprime(n) and residue_lower_half(n, j))
        implies count_lower_half(n, j.suc) = count_lower_half(n, j)
}

/// Recurrence: a coprime upper-half index adds one to the upper-half count.
theorem count_upper_half_suc_yes(n: Nat, j: Nat) {
    j.coprime(n) and residue_upper_half(n, j)
        implies count_upper_half(n, j.suc) = count_upper_half(n, j) + Nat.1
}

/// Recurrence: an index that is not a coprime upper-half element leaves the
/// upper-half count alone.
theorem count_upper_half_suc_no(n: Nat, j: Nat) {
    not (j.coprime(n) and residue_upper_half(n, j))
        implies count_upper_half(n, j.suc) = count_upper_half(n, j)
}

/// Inductive predicate for the coprime/upper/lower partition law.
define coprime_half_partition_pred(n: Nat) -> (Nat -> Bool) {
    function(k: Nat) {
        count_coprime_to(n, k) = count_lower_half(n, k) + count_upper_half(n, k)
    }
}

/// Inductive step: a coprime `j` falls in exactly one half, so the two sides
/// advance together.
theorem coprime_half_partition_step(n: Nat, j: Nat) {
    Nat.2 < n and coprime_half_partition_pred(n)(j)
        implies coprime_half_partition_pred(n)(j.suc)
} by {
    if Nat.2 < n and coprime_half_partition_pred(n)(j) {
        count_coprime_to(n, j) = count_lower_half(n, j) + count_upper_half(n, j)
        if j.coprime(n) {
            count_coprime_to_suc_yes(n, j)
            count_coprime_to(n, j.suc) = count_coprime_to(n, j) + Nat.1
            residue_lower_half(n, j) = (j + j < n)
            residue_upper_half(n, j) = (n < j + j)
            lt_or_lte(j + j, n)
            if j + j < n {
                count_lower_half_suc_yes(n, j)
                count_lower_half(n, j.suc) = count_lower_half(n, j) + Nat.1
                if n < j + j {
                    lt_trans(j + j, n, j + j)
                    j + j < j + j
                    false
                }
                not (n < j + j)
                not residue_upper_half(n, j)
                if j.coprime(n) and residue_upper_half(n, j) {
                    false
                }
                not (j.coprime(n) and residue_upper_half(n, j))
                count_upper_half_suc_no(n, j)
                count_upper_half(n, j.suc) = count_upper_half(n, j)
                count_coprime_to(n, j) = count_lower_half(n, j) + count_upper_half(n, j)
                add_assoc(count_lower_half(n, j), Nat.1, count_upper_half(n, j))
                (count_lower_half(n, j) + Nat.1) + count_upper_half(n, j) =
                    count_lower_half(n, j) + (Nat.1 + count_upper_half(n, j))
                add_comm(Nat.1, count_upper_half(n, j))
                Nat.1 + count_upper_half(n, j) = count_upper_half(n, j) + Nat.1
                count_lower_half(n, j) + (Nat.1 + count_upper_half(n, j)) =
                    count_lower_half(n, j) + (count_upper_half(n, j) + Nat.1)
                add_assoc(count_lower_half(n, j), count_upper_half(n, j), Nat.1)
                count_lower_half(n, j) + count_upper_half(n, j) + Nat.1 =
                    count_lower_half(n, j) + (count_upper_half(n, j) + Nat.1)
                (count_lower_half(n, j) + Nat.1) + count_upper_half(n, j) =
                    count_lower_half(n, j) + count_upper_half(n, j) + Nat.1
                count_coprime_to(n, j.suc) =
                    count_lower_half(n, j.suc) + count_upper_half(n, j.suc)
            }
            if not (j + j < n) {
                coprime_not_half(n, j)
                j + j != n
                not_lt_imp_lte(j + j, n)
                n <= j + j
                if n = j + j {
                    j + j = n
                    false
                }
                n != j + j
                lte_neq_imp_lt(n, j + j)
                n < j + j
                count_upper_half_suc_yes(n, j)
                count_upper_half(n, j.suc) = count_upper_half(n, j) + Nat.1
                if j + j < n {
                    lt_trans(j + j, n, j + j)
                    j + j < j + j
                    false
                }
                not (j + j < n)
                not residue_lower_half(n, j)
                if j.coprime(n) and residue_lower_half(n, j) {
                    false
                }
                not (j.coprime(n) and residue_lower_half(n, j))
                count_lower_half_suc_no(n, j)
                count_lower_half(n, j.suc) = count_lower_half(n, j)
                count_coprime_to(n, j) = count_lower_half(n, j) + count_upper_half(n, j)
                add_assoc(count_lower_half(n, j), count_upper_half(n, j), Nat.1)
                count_lower_half(n, j) + count_upper_half(n, j) + Nat.1 =
                    count_lower_half(n, j) + (count_upper_half(n, j) + Nat.1)
                count_coprime_to(n, j.suc) =
                    count_lower_half(n, j.suc) + count_upper_half(n, j.suc)
            }
            count_coprime_to(n, j.suc) =
                count_lower_half(n, j.suc) + count_upper_half(n, j.suc)
        } else {
            not j.coprime(n)
            count_coprime_to_suc_no(n, j)
            count_coprime_to(n, j.suc) = count_coprime_to(n, j)
            count_lower_half_suc_no(n, j)
            count_lower_half(n, j.suc) = count_lower_half(n, j)
            count_upper_half_suc_no(n, j)
            count_upper_half(n, j.suc) = count_upper_half(n, j)
            count_coprime_to(n, j.suc) =
                count_lower_half(n, j.suc) + count_upper_half(n, j.suc)
        }
        coprime_half_partition_pred(n)(j.suc) =
            (count_coprime_to(n, j.suc) = count_lower_half(n, j.suc) + count_upper_half(n, j.suc))
        coprime_half_partition_pred(n)(j.suc)
    }
}

/// Walking the partition law up to `k`.
theorem coprime_half_partition_run(n: Nat, k: Nat) {
    Nat.2 < n implies coprime_half_partition_pred(n)(k)
} by {
    if Nat.2 < n {
        let f: Nat -> Bool = function(x: Nat) {
            count_coprime_to(n, x) = count_lower_half(n, x) + count_upper_half(n, x)
        }
        forall(y: Nat) {
            coprime_half_partition_pred(n)(y) = f(y)
            f(y) = coprime_half_partition_pred(n)(y)
        }
        count_coprime_to(n, Nat.0) = Nat.0
        count_lower_half(n, Nat.0) = Nat.0
        count_upper_half(n, Nat.0) = Nat.0
        f(Nat.0)
        forall(x: Nat) {
            if f(x) {
                coprime_half_partition_pred(n)(x)
                coprime_half_partition_step(n, x)
                coprime_half_partition_pred(n)(x.suc)
                f(x.suc)
            }
        }
        forall(x: Nat) {
            f(x) implies f(x.suc)
        }
        f(Nat.0) and forall(x: Nat) {
            f(x) implies f(x.suc)
        }
        Nat.induction(f)
        forall(k0: Nat) { f(k0) }
        f(k)
        coprime_half_partition_pred(n)(k)
    }
}

/// The coprime counts split into the lower and upper halves:
/// `count_coprime_to(n, k) = count_lower_half(n, k) + count_upper_half(n, k)`.
theorem count_coprime_partition(n: Nat, k: Nat) {
    Nat.2 < n implies
        count_coprime_to(n, k) = count_lower_half(n, k) + count_upper_half(n, k)
} by {
    if Nat.2 < n {
        coprime_half_partition_run(n, k)
        coprime_half_partition_pred(n)(k)
    }
}

/// Inductive predicate for the lower-half list-count identity.
define lower_half_count_pred(n: Nat) -> (Nat -> Bool) {
    function(k: Nat) {
        coprime_residues_below(n, k).filter(residue_lower_half(n)).length = count_lower_half(n, k)
    }
}

/// Inductive step: the residue list and the count grow together at a
/// lower-half coprime index.
theorem lower_half_count_step(n: Nat, j: Nat) {
    Nat.2 < n and lower_half_count_pred(n)(j)
        implies lower_half_count_pred(n)(j.suc)
} by {
    if Nat.2 < n and lower_half_count_pred(n)(j) {
        coprime_residues_below(n, j).filter(residue_lower_half(n)).length = count_lower_half(n, j)
        if j.coprime(n) {
            coprime_residues_below_suc_yes(n, j)
            coprime_residues_below(n, j.suc) = List.cons(j, coprime_residues_below(n, j))
            if residue_lower_half(n, j) {
                List.cons(j, coprime_residues_below(n, j)).filter(residue_lower_half(n)) =
                    List.cons(j, coprime_residues_below(n, j).filter(residue_lower_half(n)))
                coprime_residues_below(n, j.suc).filter(residue_lower_half(n)) =
                    List.cons(j, coprime_residues_below(n, j).filter(residue_lower_half(n)))
                List.cons(j, coprime_residues_below(n, j).filter(residue_lower_half(n))).length =
                    coprime_residues_below(n, j).filter(residue_lower_half(n)).length.suc
                count_lower_half_suc_yes(n, j)
                count_lower_half(n, j.suc) = count_lower_half(n, j) + Nat.1
                coprime_residues_below(n, j).filter(residue_lower_half(n)).length + Nat.1 =
                    count_lower_half(n, j) + Nat.1
                coprime_residues_below(n, j.suc).filter(residue_lower_half(n)).length =
                    count_lower_half(n, j.suc)
            } else {
                not residue_lower_half(n, j)
                List.cons(j, coprime_residues_below(n, j)).filter(residue_lower_half(n)) =
                    coprime_residues_below(n, j).filter(residue_lower_half(n))
                coprime_residues_below(n, j.suc).filter(residue_lower_half(n)) =
                    coprime_residues_below(n, j).filter(residue_lower_half(n))
                if j.coprime(n) and residue_lower_half(n, j) {
                    false
                }
                not (j.coprime(n) and residue_lower_half(n, j))
                count_lower_half_suc_no(n, j)
                count_lower_half(n, j.suc) = count_lower_half(n, j)
                coprime_residues_below(n, j.suc).filter(residue_lower_half(n)).length =
                    count_lower_half(n, j.suc)
            }
        } else {
            not j.coprime(n)
            coprime_residues_below_suc_no(n, j)
            coprime_residues_below(n, j.suc) = coprime_residues_below(n, j)
            if j.coprime(n) and residue_lower_half(n, j) {
                false
            }
            not (j.coprime(n) and residue_lower_half(n, j))
            count_lower_half_suc_no(n, j)
            count_lower_half(n, j.suc) = count_lower_half(n, j)
            coprime_residues_below(n, j.suc).filter(residue_lower_half(n)).length =
                count_lower_half(n, j.suc)
        }
        lower_half_count_pred(n)(j.suc) =
            (coprime_residues_below(n, j.suc).filter(residue_lower_half(n)).length = count_lower_half(n, j.suc))
        lower_half_count_pred(n)(j.suc)
    }
}

/// Walking the lower-half list-count identity up to `k`.
theorem lower_half_count_run(n: Nat, k: Nat) {
    Nat.2 < n implies lower_half_count_pred(n)(k)
} by {
    if Nat.2 < n {
        let f: Nat -> Bool = function(x: Nat) {
            coprime_residues_below(n, x).filter(residue_lower_half(n)).length = count_lower_half(n, x)
        }
        forall(y: Nat) {
            lower_half_count_pred(n)(y) = f(y)
            f(y) = lower_half_count_pred(n)(y)
        }
        coprime_residues_below(n, Nat.0) = List.nil[Nat]
        List.nil[Nat].filter(residue_lower_half(n)) = List.nil[Nat]
        List.nil[Nat].filter(residue_lower_half(n)).length = Nat.0
        count_lower_half(n, Nat.0) = Nat.0
        f(Nat.0)
        forall(x: Nat) {
            if f(x) {
                lower_half_count_pred(n)(x)
                lower_half_count_step(n, x)
                lower_half_count_pred(n)(x.suc)
                f(x.suc)
            }
        }
        forall(x: Nat) {
            f(x) implies f(x.suc)
        }
        f(Nat.0) and forall(x: Nat) {
            f(x) implies f(x.suc)
        }
        Nat.induction(f)
        forall(k0: Nat) { f(k0) }
        f(k)
        lower_half_count_pred(n)(k)
    }
}

/// The lower-half residues list has length `count_lower_half(n, n)`.
theorem lower_half_residues_length_eq_count(n: Nat) {
    Nat.2 < n implies lower_half_residues(n).length = count_lower_half(n, n)
} by {
    if Nat.2 < n {
        lower_half_count_run(n, n)
        lower_half_count_pred(n)(n)
        coprime_residues_below(n, n).filter(residue_lower_half(n)).length = count_lower_half(n, n)
        coprime_residues(n) = coprime_residues_below(n, n)
        coprime_residues(n).filter(residue_lower_half(n)).length = count_lower_half(n, n)
        lower_half_residues(n) = coprime_residues(n).filter(residue_lower_half(n))
        lower_half_residues(n).length = count_lower_half(n, n)
    }
}

/// Inductive predicate for the upper-half list-count identity.
define upper_half_count_pred(n: Nat) -> (Nat -> Bool) {
    function(k: Nat) {
        coprime_residues_below(n, k).filter(residue_upper_half(n)).length = count_upper_half(n, k)
    }
}

/// Inductive step: the residue list and the count grow together at an
/// upper-half coprime index.
theorem upper_half_count_step(n: Nat, j: Nat) {
    Nat.2 < n and upper_half_count_pred(n)(j)
        implies upper_half_count_pred(n)(j.suc)
} by {
    if Nat.2 < n and upper_half_count_pred(n)(j) {
        coprime_residues_below(n, j).filter(residue_upper_half(n)).length = count_upper_half(n, j)
        if j.coprime(n) {
            coprime_residues_below_suc_yes(n, j)
            coprime_residues_below(n, j.suc) = List.cons(j, coprime_residues_below(n, j))
            if residue_upper_half(n, j) {
                List.cons(j, coprime_residues_below(n, j)).filter(residue_upper_half(n)) =
                    List.cons(j, coprime_residues_below(n, j).filter(residue_upper_half(n)))
                coprime_residues_below(n, j.suc).filter(residue_upper_half(n)) =
                    List.cons(j, coprime_residues_below(n, j).filter(residue_upper_half(n)))
                List.cons(j, coprime_residues_below(n, j).filter(residue_upper_half(n))).length =
                    coprime_residues_below(n, j).filter(residue_upper_half(n)).length.suc
                count_upper_half_suc_yes(n, j)
                count_upper_half(n, j.suc) = count_upper_half(n, j) + Nat.1
                coprime_residues_below(n, j).filter(residue_upper_half(n)).length + Nat.1 =
                    count_upper_half(n, j) + Nat.1
                coprime_residues_below(n, j.suc).filter(residue_upper_half(n)).length =
                    count_upper_half(n, j.suc)
            } else {
                not residue_upper_half(n, j)
                List.cons(j, coprime_residues_below(n, j)).filter(residue_upper_half(n)) =
                    coprime_residues_below(n, j).filter(residue_upper_half(n))
                coprime_residues_below(n, j.suc).filter(residue_upper_half(n)) =
                    coprime_residues_below(n, j).filter(residue_upper_half(n))
                if j.coprime(n) and residue_upper_half(n, j) {
                    false
                }
                not (j.coprime(n) and residue_upper_half(n, j))
                count_upper_half_suc_no(n, j)
                count_upper_half(n, j.suc) = count_upper_half(n, j)
                coprime_residues_below(n, j.suc).filter(residue_upper_half(n)).length =
                    count_upper_half(n, j.suc)
            }
        } else {
            not j.coprime(n)
            coprime_residues_below_suc_no(n, j)
            coprime_residues_below(n, j.suc) = coprime_residues_below(n, j)
            if j.coprime(n) and residue_upper_half(n, j) {
                false
            }
            not (j.coprime(n) and residue_upper_half(n, j))
            count_upper_half_suc_no(n, j)
            count_upper_half(n, j.suc) = count_upper_half(n, j)
            coprime_residues_below(n, j.suc).filter(residue_upper_half(n)).length =
                count_upper_half(n, j.suc)
        }
        upper_half_count_pred(n)(j.suc) =
            (coprime_residues_below(n, j.suc).filter(residue_upper_half(n)).length = count_upper_half(n, j.suc))
        upper_half_count_pred(n)(j.suc)
    }
}

/// Walking the upper-half list-count identity up to `k`.
theorem upper_half_count_run(n: Nat, k: Nat) {
    Nat.2 < n implies upper_half_count_pred(n)(k)
} by {
    if Nat.2 < n {
        let f: Nat -> Bool = function(x: Nat) {
            coprime_residues_below(n, x).filter(residue_upper_half(n)).length = count_upper_half(n, x)
        }
        forall(y: Nat) {
            upper_half_count_pred(n)(y) = f(y)
            f(y) = upper_half_count_pred(n)(y)
        }
        coprime_residues_below(n, Nat.0) = List.nil[Nat]
        List.nil[Nat].filter(residue_upper_half(n)) = List.nil[Nat]
        List.nil[Nat].filter(residue_upper_half(n)).length = Nat.0
        count_upper_half(n, Nat.0) = Nat.0
        f(Nat.0)
        forall(x: Nat) {
            if f(x) {
                upper_half_count_pred(n)(x)
                upper_half_count_step(n, x)
                upper_half_count_pred(n)(x.suc)
                f(x.suc)
            }
        }
        forall(x: Nat) {
            f(x) implies f(x.suc)
        }
        f(Nat.0) and forall(x: Nat) {
            f(x) implies f(x.suc)
        }
        Nat.induction(f)
        forall(k0: Nat) { f(k0) }
        f(k)
        upper_half_count_pred(n)(k)
    }
}

/// The upper-half residues list has length `count_upper_half(n, n)`.
theorem upper_half_residues_length_eq_count(n: Nat) {
    Nat.2 < n implies upper_half_residues(n).length = count_upper_half(n, n)
} by {
    if Nat.2 < n {
        upper_half_count_run(n, n)
        upper_half_count_pred(n)(n)
        coprime_residues_below(n, n).filter(residue_upper_half(n)).length = count_upper_half(n, n)
        coprime_residues(n) = coprime_residues_below(n, n)
        coprime_residues(n).filter(residue_upper_half(n)).length = count_upper_half(n, n)
        upper_half_residues(n) = coprime_residues(n).filter(residue_upper_half(n))
        upper_half_residues(n).length = count_upper_half(n, n)
    }
}

/// Reflection carries a lower-half reduced residue into an upper-half one.
theorem lower_reflect_mem_upper(n: Nat, y: Nat) {
    Nat.2 < n and lower_half_residues(n).contains(y)
        implies upper_half_residues(n).contains(reflect_residue(n, y))
} by {
    if Nat.2 < n and lower_half_residues(n).contains(y) {
        lower_half_residues(n) = coprime_residues(n).filter(residue_lower_half(n))
        coprime_residues(n).filter(residue_lower_half(n)).contains(y)
        filter_contained_by_and(coprime_residues(n), residue_lower_half(n), y)
        coprime_residues(n).contains(y) and residue_lower_half(n, y)
        coprime_residues(n).contains(y)
        residue_lower_half(n, y)
        coprime_residues_contains_imp(n, y)
        y < n and y.coprime(n)
        y < n
        y.coprime(n)
        residue_nonzero(n, y)
        y != Nat.0
        pos_of_ne_zero(y)
        Nat.0 < y
        reflect_residue_in_range(n, y)
        Nat.0 < reflect_residue(n, y) and reflect_residue(n, y) < n
        reflect_residue(n, y) < n
        reflect_residue_coprime(n, y)
        reflect_residue(n, y).coprime(n)
        coprime_residues_contains_intro(n, reflect_residue(n, y))
        coprime_residues(n).contains(reflect_residue(n, y))
        residue_lower_half(n, y) = (y + y < n)
        y + y < n
        reflect_residue_adds_to_modulus(n, y)
        y + reflect_residue(n, y) = n
        add_lt_cancel_left(y, y, reflect_residue(n, y))
        y < reflect_residue(n, y)
        lt_add_left(reflect_residue(n, y), y, reflect_residue(n, y))
        reflect_residue(n, y) + y < reflect_residue(n, y) + reflect_residue(n, y)
        add_comm(reflect_residue(n, y), y)
        reflect_residue(n, y) + y = y + reflect_residue(n, y)
        y + reflect_residue(n, y) = n
        reflect_residue(n, y) + y = n
        n < reflect_residue(n, y) + reflect_residue(n, y)
        residue_upper_half(n, reflect_residue(n, y)) =
            (n < reflect_residue(n, y) + reflect_residue(n, y))
        residue_upper_half(n, reflect_residue(n, y))
        coprime_residues(n).contains(reflect_residue(n, y)) and
            residue_upper_half(n, reflect_residue(n, y))
        filter_equivalent_to_and(coprime_residues(n), residue_upper_half(n), reflect_residue(n, y))
        (coprime_residues(n).contains(reflect_residue(n, y)) and
            residue_upper_half(n, reflect_residue(n, y))) =
            coprime_residues(n).filter(residue_upper_half(n)).contains(reflect_residue(n, y))
        coprime_residues(n).filter(residue_upper_half(n)).contains(reflect_residue(n, y))
        upper_half_residues(n) = coprime_residues(n).filter(residue_upper_half(n))
        upper_half_residues(n).contains(reflect_residue(n, y))
    }
}

/// Reflection carries an upper-half reduced residue into a lower-half one.
theorem upper_reflect_mem_lower(n: Nat, y: Nat) {
    Nat.2 < n and upper_half_residues(n).contains(y)
        implies lower_half_residues(n).contains(reflect_residue(n, y))
} by {
    if Nat.2 < n and upper_half_residues(n).contains(y) {
        upper_half_residues(n) = coprime_residues(n).filter(residue_upper_half(n))
        coprime_residues(n).filter(residue_upper_half(n)).contains(y)
        filter_contained_by_and(coprime_residues(n), residue_upper_half(n), y)
        coprime_residues(n).contains(y) and residue_upper_half(n, y)
        coprime_residues(n).contains(y)
        residue_upper_half(n, y)
        coprime_residues_contains_imp(n, y)
        y < n and y.coprime(n)
        y < n
        y.coprime(n)
        residue_nonzero(n, y)
        y != Nat.0
        pos_of_ne_zero(y)
        Nat.0 < y
        reflect_residue_in_range(n, y)
        Nat.0 < reflect_residue(n, y) and reflect_residue(n, y) < n
        reflect_residue(n, y) < n
        reflect_residue_coprime(n, y)
        reflect_residue(n, y).coprime(n)
        coprime_residues_contains_intro(n, reflect_residue(n, y))
        coprime_residues(n).contains(reflect_residue(n, y))
        residue_upper_half(n, y) = (n < y + y)
        n < y + y
        reflect_residue_adds_to_modulus(n, y)
        y + reflect_residue(n, y) = n
        add_lt_cancel_left(y, reflect_residue(n, y), y)
        reflect_residue(n, y) < y
        lt_add_left(reflect_residue(n, y), reflect_residue(n, y), y)
        reflect_residue(n, y) + reflect_residue(n, y) < reflect_residue(n, y) + y
        add_comm(reflect_residue(n, y), y)
        reflect_residue(n, y) + y = y + reflect_residue(n, y)
        y + reflect_residue(n, y) = n
        reflect_residue(n, y) + y = n
        reflect_residue(n, y) + reflect_residue(n, y) < n
        residue_lower_half(n, reflect_residue(n, y)) =
            (reflect_residue(n, y) + reflect_residue(n, y) < n)
        residue_lower_half(n, reflect_residue(n, y))
        coprime_residues(n).contains(reflect_residue(n, y)) and
            residue_lower_half(n, reflect_residue(n, y))
        filter_equivalent_to_and(coprime_residues(n), residue_lower_half(n), reflect_residue(n, y))
        (coprime_residues(n).contains(reflect_residue(n, y)) and
            residue_lower_half(n, reflect_residue(n, y))) =
            coprime_residues(n).filter(residue_lower_half(n)).contains(reflect_residue(n, y))
        coprime_residues(n).filter(residue_lower_half(n)).contains(reflect_residue(n, y))
        lower_half_residues(n) = coprime_residues(n).filter(residue_lower_half(n))
        lower_half_residues(n).contains(reflect_residue(n, y))
    }
}

/// The reflection `a -> n - a` pairs the lower-half reduced residues with the
/// upper-half ones: the mapped lower list is a permutation of the upper list.
theorem reflect_pairs_halves(n: Nat) {
    Nat.2 < n implies
        is_permutation(map(lower_half_residues(n), reflect_residue(n)), upper_half_residues(n))
} by {
    if Nat.2 < n {
        forall(y: Nat) {
            if map(lower_half_residues(n), reflect_residue(n)).contains(y) {
                map_contains(lower_half_residues(n), reflect_residue(n), y)
                let x: Nat satisfy {
                    lower_half_residues(n).contains(x) and reflect_residue(n, x) = y
                }
                lower_reflect_mem_upper(n, x)
                upper_half_residues(n).contains(reflect_residue(n, x))
                upper_half_residues(n).contains(y)
            }
            if upper_half_residues(n).contains(y) {
                upper_reflect_mem_lower(n, y)
                lower_half_residues(n).contains(reflect_residue(n, y))
                filter_contained_by_and(coprime_residues(n), residue_upper_half(n), y)
                coprime_residues(n).contains(y)
                coprime_residues_contains_imp(n, y)
                y < n and y.coprime(n)
                y < n
                reflect_residue_involution(n, y)
                reflect_residue(n, reflect_residue(n, y)) = y
                map_contains_of_contains(lower_half_residues(n), reflect_residue(n),
                    reflect_residue(n, y))
                map(lower_half_residues(n), reflect_residue(n)).contains(
                    reflect_residue(n, reflect_residue(n, y)))
                map(lower_half_residues(n), reflect_residue(n)).contains(y)
            }
            map(lower_half_residues(n), reflect_residue(n)).contains(y) =
                upper_half_residues(n).contains(y)
        }
        coprime_residues_unique(n)
        coprime_residues(n).is_unique
        filter_preserves_unique[Nat](coprime_residues(n), residue_lower_half(n))
        lower_half_residues(n).is_unique
        filter_preserves_unique[Nat](coprime_residues(n), residue_upper_half(n))
        upper_half_residues(n).is_unique
        forall(a: Nat, b: Nat) {
            if lower_half_residues(n).contains(a) and lower_half_residues(n).contains(b)
                and reflect_residue(n, a) = reflect_residue(n, b) {
                filter_contained_by_and(coprime_residues(n), residue_lower_half(n), a)
                coprime_residues(n).contains(a)
                coprime_residues_contains_imp(n, a)
                a < n and a.coprime(n)
                a < n
                filter_contained_by_and(coprime_residues(n), residue_lower_half(n), b)
                coprime_residues(n).contains(b)
                coprime_residues_contains_imp(n, b)
                b < n and b.coprime(n)
                b < n
                reflect_residue_involution(n, a)
                reflect_residue(n, reflect_residue(n, a)) = a
                reflect_residue_involution(n, b)
                reflect_residue(n, reflect_residue(n, b)) = b
                reflect_residue(n, a) = reflect_residue(n, b)
                reflect_residue(n, reflect_residue(n, a)) = reflect_residue(n, reflect_residue(n, b))
                a = b
            }
        }
        locally_injective_map_is_unique(lower_half_residues(n), reflect_residue(n))
        map(lower_half_residues(n), reflect_residue(n)).is_unique
        unique_same_contains_imp_permutation(
            map(lower_half_residues(n), reflect_residue(n)), upper_half_residues(n))
        is_permutation(map(lower_half_residues(n), reflect_residue(n)), upper_half_residues(n))
    }
}

/// The upper-half reduced residues are exactly as many as the lower-half ones.
theorem upper_half_length_eq_lower_half(n: Nat) {
    Nat.2 < n implies upper_half_residues(n).length = lower_half_residues(n).length
} by {
    if Nat.2 < n {
        reflect_pairs_halves(n)
        is_permutation(map(lower_half_residues(n), reflect_residue(n)), upper_half_residues(n))
        permutation_preserves_length(
            map(lower_half_residues(n), reflect_residue(n)), upper_half_residues(n))
        map(lower_half_residues(n), reflect_residue(n)).length = upper_half_residues(n).length
        map_length(lower_half_residues(n), reflect_residue(n))
        map(lower_half_residues(n), reflect_residue(n)).length = lower_half_residues(n).length
        upper_half_residues(n).length = lower_half_residues(n).length
    }
}

/// Euler's totient is even for `n > 2`: the reflection pairs the reduced
/// residues without fixed points, so they split into two halves of equal size.
/// Stated with the explicit double: `totient(n) = 2 * |lower half|`.
theorem totient_even_double(n: Nat) {
    Nat.2 < n implies n.totient = Nat.2 * lower_half_residues(n).length
} by {
    if Nat.2 < n {
        n.totient = count_coprime_to(n, n)
        count_coprime_partition(n, n)
        count_coprime_to(n, n) = count_lower_half(n, n) + count_upper_half(n, n)
        lower_half_residues_length_eq_count(n)
        lower_half_residues(n).length = count_lower_half(n, n)
        upper_half_residues_length_eq_count(n)
        upper_half_residues(n).length = count_upper_half(n, n)
        upper_half_length_eq_lower_half(n)
        upper_half_residues(n).length = lower_half_residues(n).length
        n.totient = lower_half_residues(n).length + upper_half_residues(n).length
        n.totient = lower_half_residues(n).length + lower_half_residues(n).length
        mul_two_left(lower_half_residues(n).length)
        Nat.2 * lower_half_residues(n).length = lower_half_residues(n).length + lower_half_residues(n).length
        n.totient = Nat.2 * lower_half_residues(n).length
    }
}

/// Euler's totient is even for `n > 2`.
theorem totient_even(n: Nat) {
    Nat.2 < n implies is_even(n.totient)
} by {
    if Nat.2 < n {
        totient_even_double(n)
        n.totient = Nat.2 * lower_half_residues(n).length
        is_even(n.totient) = exists(k: Nat) { n.totient = Nat.2 * k }
        exists(k: Nat) { n.totient = Nat.2 * k }
        is_even(n.totient)
    }
}

/// Half-modulus characterisation, forward direction: for `n = 2^k` with
/// `k >= 1`, `phi(n) = n / 2`, stated without division as
/// `totient(2^k) * 2 = 2^k`.
theorem totient_power_of_two_half(k: Nat) {
    Nat.1 <= k implies (Nat.2.pow(k)).totient * Nat.2 = Nat.2.pow(k)
} by {
    if Nat.1 <= k {
        zero_or_suc(k)
        if k = Nat.0 {
            not (Nat.1 <= Nat.0)
            Nat.1 <= Nat.0
            false
        }
        let q: Nat satisfy { k = q.suc }
        k = q.suc
        add_imp_sub(q, Nat.1, k)
        k - Nat.1 = q
        two_is_prime
        Nat.2.is_prime
        totient_p_pow_factored(Nat.2, q)
        (Nat.2.pow(q.suc)).totient = (Nat.2 - Nat.1) * Nat.2.pow(q)
        Nat.2.pow(q.suc) = Nat.2.pow(k)
        (Nat.2.pow(k)).totient = (Nat.2 - Nat.1) * Nat.2.pow(q)
        Nat.2 - Nat.1 = Nat.1
        (Nat.2 - Nat.1) * Nat.2.pow(q) = Nat.1 * Nat.2.pow(q)
        Nat.1 * Nat.2.pow(q) = Nat.2.pow(q)
        (Nat.2.pow(k)).totient = Nat.2.pow(q)
        (Nat.2.pow(k)).totient = Nat.2.pow(k - Nat.1)
        exp_add(Nat.2, q, Nat.1)
        Nat.2.pow(q + Nat.1) = Nat.2.pow(q) * Nat.2.pow(Nat.1)
        exp_one(Nat.2)
        Nat.2.pow(Nat.1) = Nat.2
        q + Nat.1 = k
        Nat.2.pow(q + Nat.1) = Nat.2.pow(k)
        Nat.2.pow(k) = Nat.2.pow(q) * Nat.2
        (Nat.2.pow(k)).totient * Nat.2 = Nat.2.pow(q) * Nat.2
        (Nat.2.pow(k)).totient * Nat.2 = Nat.2.pow(k)
    }
}

// The full characterisation `phi(n) = n / 2` iff `n = 2^k` needs the converse:
// if `n` has an odd prime factor `p`, then the product formula gives
// `phi(n) = n * prod_{q | n} (1 - 1/q) <= n * (1/2) * (2/3) < n / 2`.  The
// forward direction (`n = 2^k` implies `phi(n) = n / 2`) is proved above as
// `totient_power_of_two_half`; the converse is left for the factorisation lane.

/// `phi(2) = 1`: the only reduced residue below two is one.
theorem totient_two {
    nat_totient(Nat.2) = Nat.1
} by {
    two_is_prime
    Nat.2.is_prime
    Nat.1 <= Nat.1
    totient_prime_power(Nat.2, Nat.1)
    (Nat.2.pow(Nat.1)).totient = Nat.2.pow(Nat.1) - Nat.2.pow(Nat.1 - Nat.1)
    Nat.1 - Nat.1 = Nat.0
    exp_one(Nat.2)
    Nat.2.pow(Nat.1) = Nat.2
    exp_zero(Nat.2)
    Nat.2.pow(Nat.0) = Nat.1
    Nat.2.pow(Nat.1) - Nat.2.pow(Nat.1 - Nat.1) = Nat.2 - Nat.1
    Nat.2 - Nat.1 = Nat.1
    (Nat.2.pow(Nat.1)).totient = Nat.1
    Nat.2.totient = Nat.1
    nat_totient(Nat.2) = Nat.2.totient
    nat_totient(Nat.2) = Nat.1
}

/// `phi(4) = 2`, the first nontrivial power of two.
theorem totient_four {
    nat_totient(Nat.4) = Nat.2
} by {
    two_is_prime
    Nat.2.is_prime
    Nat.1 <= Nat.2
    totient_prime_power(Nat.2, Nat.2)
    (Nat.2.pow(Nat.2)).totient = Nat.2.pow(Nat.2) - Nat.2.pow(Nat.2 - Nat.1)
    Nat.2 - Nat.1 = Nat.1
    Nat.2.pow(Nat.2) = Nat.4
    exp_one(Nat.2)
    Nat.2.pow(Nat.1) = Nat.2
    Nat.2.pow(Nat.2) - Nat.2.pow(Nat.1) = Nat.4 - Nat.2
    Nat.4 - Nat.2 = Nat.2
    (Nat.2.pow(Nat.2)).totient = Nat.2
    Nat.4.totient = Nat.2
    nat_totient(Nat.4) = Nat.4.totient
    nat_totient(Nat.4) = Nat.2
}

/// `phi(8) = 4`, the second nontrivial power of two.
theorem totient_eight {
    nat_totient(Nat.8) = Nat.4
} by {
    two_is_prime
    Nat.2.is_prime
    Nat.1 <= Nat.3
    totient_prime_power(Nat.2, Nat.3)
    (Nat.2.pow(Nat.3)).totient = Nat.2.pow(Nat.3) - Nat.2.pow(Nat.3 - Nat.1)
    Nat.3 - Nat.1 = Nat.2
    Nat.2.pow(Nat.2) = Nat.4
    exp_add(Nat.2, Nat.1, Nat.2)
    Nat.2.pow(Nat.1 + Nat.2) = Nat.2.pow(Nat.1) * Nat.2.pow(Nat.2)
    Nat.1 + Nat.2 = Nat.3
    Nat.2.pow(Nat.1 + Nat.2) = Nat.2.pow(Nat.3)
    Nat.2.pow(Nat.3) = Nat.2.pow(Nat.1) * Nat.2.pow(Nat.2)
    exp_one(Nat.2)
    Nat.2.pow(Nat.1) = Nat.2
    Nat.2.pow(Nat.1) * Nat.2.pow(Nat.2) = Nat.2 * Nat.4
    Nat.2 * Nat.4 = Nat.8
    Nat.2.pow(Nat.3) = Nat.8
    Nat.2.pow(Nat.3) - Nat.2.pow(Nat.2) = Nat.8 - Nat.4
    Nat.4 + Nat.4 = Nat.8
    add_imp_sub(Nat.4, Nat.4, Nat.8)
    Nat.8 - Nat.4 = Nat.4
    (Nat.2.pow(Nat.3)).totient = Nat.4
    Nat.8.totient = Nat.4
    nat_totient(Nat.8) = Nat.8.totient
    nat_totient(Nat.8) = Nat.4
}
