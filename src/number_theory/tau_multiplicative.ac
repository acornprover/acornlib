from nat import Nat
from nat import divides_self, divides_lte, mul_cancel_left,
    gcd_divides_left, gcd_divides_right, lte_antisymm, lte_trans,
    lte_ref, lt_not_ref, not_lt_zero, lt_suc_right,
    lt_imp_lte_suc, lte_and_lt, lt_and_lte, lte_imp_not_lt, trichotomy,
    pos_of_ne_zero, lt_diff, lt_mul_both, lte_mul, lte_mul_both,
    exp_zero, exp_add, exp_ne_zero, exp_one
from list import List, map, length_range, range_contains_of_lt, lt_of_range_contains,
    range_is_unique, map_length, map_contains, map_contains_of_contains,
    unique_is_smallest_containing_list, singleton_contains_imp_eq
from data.finite.finite_fiber_partition import locally_injective_map_is_unique
from number_theory.divisor_sum import divisor_list, divisors_up_to,
    divisors_up_to_complete, nat_tau, nat_tau_zero, nat_tau_one, nat_tau_prime,
    divisor_list_contains_implies, divisor_list_contains_of, divisor_list_is_unique,
    one_divides_nat
from number_theory.coprime import coprime_divides_of_divides_mul, nat_divides_one_imp_one
from number_theory.factorisation import prime_divisor_is_one_or_self, no_proper_divisor_imp_prime
from number_theory.arithmetic_functions import is_multiplicative_nat_fn, multiplicative_nat_fn_apply
from number_theory.dirichlet import nat_tau_multiplicative
numerals Nat

// ---------------------------------------------------------------------------
// Power (exponentiation) monotonicity lemmas.
//
// The `nat` module exposes the power laws (`exp_zero`, `exp_add`,
// `exp_ne_zero`) but not the monotonicity statements; they are developed here.
// ---------------------------------------------------------------------------

/// A positive base has power at least one.
theorem exp_gte_one(a: Nat, b: Nat) {
    a != Nat.0 implies Nat.1 <= a.pow(b)
} by {
    let f: Nat -> Bool = function(x: Nat) { Nat.1 <= a.pow(x) }
    exp_zero(a)
    a.pow(Nat.0) = Nat.1
    f(Nat.0)
    forall(x: Nat) {
        if f(x) {
            a.pow(x.suc) = a * a.pow(x)
            lte_mul(a.pow(x), a)
            a.pow(x) <= a.pow(x) * a
            a.pow(x) * a = a * a.pow(x)
            a.pow(x) <= a * a.pow(x)
            a.pow(x) <= a.pow(x.suc)
            lte_trans(Nat.1, a.pow(x), a.pow(x.suc))
            Nat.1 <= a.pow(x.suc)
            f(x.suc)
        }
    }
    f(b)
}

/// A power of a number greater than one is greater than one.
theorem exp_gt_one(a: Nat, b: Nat) {
    Nat.1 < a and b != Nat.0 implies Nat.1 < a.pow(b)
} by {
    if Nat.1 < a and b != Nat.0 {
        let b_pred: Nat satisfy { b = b_pred.suc }
        b = b_pred.suc
        a.pow(b) = a.pow(b_pred.suc)
        a.pow(b_pred.suc) = a * a.pow(b_pred)
        a.pow(b) = a * a.pow(b_pred)
        a != Nat.0
        exp_gte_one(a, b_pred)
        Nat.1 <= a.pow(b_pred)
        exp_ne_zero(a, b_pred)
        a.pow(b_pred) != Nat.0
        lte_mul(a, a.pow(b_pred))
        a <= a * a.pow(b_pred)
        lt_and_lte(Nat.1, a, a * a.pow(b_pred))
        Nat.1 < a * a.pow(b_pred)
        Nat.1 < a.pow(b)
    }
}

/// Raising to a power is monotone in the exponent for nonzero bases.
theorem lte_imp_exp_lte(a: Nat, b: Nat, c: Nat) {
    a != Nat.0 and b <= c implies a.pow(b) <= a.pow(c)
} by {
    if a != Nat.0 and b <= c {
        let d: Nat satisfy { b + d = c }
        exp_ne_zero(a, d)
        a.pow(d) != Nat.0
        exp_add(a, b, d)
        a.pow(b + d) = a.pow(b) * a.pow(d)
        a.pow(b) * a.pow(d) = a.pow(b + d)
        lte_mul(a.pow(b), a.pow(d))
        a.pow(b) <= a.pow(b) * a.pow(d)
        a.pow(b) <= a.pow(b + d)
        a.pow(b + d) = a.pow(c)
        a.pow(b) <= a.pow(c)
    }
}

/// A power of a number greater than one is strictly monotone in the exponent.
theorem lt_imp_exp_lt(a: Nat, b: Nat, c: Nat) {
    Nat.1 < a and b < c implies a.pow(b) < a.pow(c)
} by {
    if Nat.1 < a and b < c {
        lt_diff(b, c)
        let d: Nat satisfy { b + d = c and d != Nat.0 }
        exp_gt_one(a, d)
        Nat.1 < a.pow(d)
        exp_add(a, b, d)
        a.pow(b + d) = a.pow(b) * a.pow(d)
        a.pow(b) * a.pow(d) = a.pow(b + d)
        a != Nat.0
        exp_ne_zero(a, b)
        a.pow(b) != Nat.0
        lt_mul_both(a.pow(b), Nat.1, a.pow(d))
        a.pow(b) * Nat.1 < a.pow(b) * a.pow(d)
        a.pow(b) * Nat.1 = a.pow(b)
        a.pow(b) < a.pow(b) * a.pow(d)
        a.pow(b) < a.pow(b + d)
        a.pow(b + d) = a.pow(c)
        a.pow(b) < a.pow(c)
    }
}

/// Monotone powers force monotone exponents for bases greater than one.
theorem exp_lte_imp_lte(a: Nat, b: Nat, c: Nat) {
    Nat.1 < a and a.pow(b) <= a.pow(c) implies b <= c
} by {
    if Nat.1 < a and a.pow(b) <= a.pow(c) {
        if not (b <= c) {
            trichotomy(b, c)
            b < c or c < b or b = c
            c < b
            lt_imp_exp_lt(a, c, b)
            a.pow(c) < a.pow(b)
            lte_imp_not_lt(a.pow(b), a.pow(c))
            false
        }
        b <= c
    }
}

/// Strictly monotone powers force strictly monotone exponents.
theorem exp_lt_imp_lt(a: Nat, b: Nat, c: Nat) {
    Nat.1 < a and a.pow(b) < a.pow(c) implies b < c
} by {
    if Nat.1 < a and a.pow(b) < a.pow(c) {
        if not (b < c) {
            trichotomy(b, c)
            b < c or c < b or b = c
            c <= b
            a != Nat.0
            lte_imp_exp_lte(a, c, b)
            a.pow(c) <= a.pow(b)
            lte_imp_not_lt(a.pow(c), a.pow(b))
            false
        }
        b < c
    }
}

/// An inequality that is not equality is strict.
theorem lte_neq_imp_lt(a: Nat, b: Nat) {
    a <= b and a != b implies a < b
} by {
    if a <= b and a != b {
        trichotomy(a, b)
        a < b or b < a or a = b
        a < b
    }
}

// ---------------------------------------------------------------------------
// Multiplicativity: tau(m n) = tau(m) * tau(n) for coprime m and n.
//
// The divisor-pairing proof (divisors of a coprime product factor uniquely as
// a divisor of each factor) lives in dirichlet.ac; the two theorems below
// restate the result in this module.
// ---------------------------------------------------------------------------

/// The divisor count `tau` is a multiplicative arithmetic function.
theorem nat_tau_is_multiplicative {
    is_multiplicative_nat_fn(nat_tau)
} by {
    nat_tau_multiplicative
}

/// `tau(m n) = tau(m) * tau(n)` whenever `m` and `n` are coprime.
theorem nat_tau_mul_coprime(m: Nat, n: Nat) {
    m.coprime(n) implies nat_tau(m * n) = nat_tau(m) * nat_tau(n)
} by {
    if m.coprime(n) {
        nat_tau_is_multiplicative
        is_multiplicative_nat_fn(nat_tau)
        multiplicative_nat_fn_apply(nat_tau, m, n)
        nat_tau(m * n) = nat_tau(m) * nat_tau(n)
    }
}

// ---------------------------------------------------------------------------
// Prime powers: tau(p^k) = k + 1.
//
// The divisors of the prime power `p^k` are exactly the powers `p^0, ..., p^k`.
// The proof has two halves.  First, every divisor of `p^k` is a power of `p`
// (by induction on `k`, splitting on `gcd(d, p)`, whose only values for prime
// `p` are `1` and `p`).  Second, the list `[p^0, ..., p^k]` and the divisor
// list of `p^k` contain each other, so the unique divisor list has the same
// length as the powers list, namely `k + 1`.
// ---------------------------------------------------------------------------

/// Base case: the only divisor of `p^0 = 1` is `1 = p^0`, a power of `p`.
theorem prime_power_divisor_base_forall(p: Nat) {
    forall(d: Nat) {
        d.divides(p.pow(Nat.0)) implies exists(i: Nat) { d = p.pow(i) and i <= Nat.0 }
    }
} by {
    forall(d: Nat) {
        if d.divides(p.pow(Nat.0)) {
            exp_zero(p)
            p.pow(Nat.0) = Nat.1
            d.divides(Nat.1)
            nat_divides_one_imp_one(d)
            d = Nat.1
            d = p.pow(Nat.0)
            Nat.0 <= Nat.0
            d = p.pow(Nat.0) and Nat.0 <= Nat.0
            exists(i: Nat) { d = p.pow(i) and i <= Nat.0 }
        }
    }
}

/// Step case: every divisor of `p^(k+1)` is a power of `p` with a bounded
/// exponent, provided the same holds at `k`.
theorem prime_power_divisor_step_forall(p: Nat, k: Nat) {
    p.is_prime and (forall(d: Nat) {
        d.divides(p.pow(k)) implies exists(i: Nat) { d = p.pow(i) and i <= k }
    }) implies forall(d: Nat) {
        d.divides(p.pow(k.suc)) implies exists(i: Nat) { d = p.pow(i) and i <= k.suc }
    }
} by {
    if p.is_prime and (forall(d: Nat) {
        d.divides(p.pow(k)) implies exists(i: Nat) { d = p.pow(i) and i <= k }
    }) {
        Nat.1 < p
        p != Nat.0
        forall(d: Nat) {
            if d.divides(p.pow(k.suc)) {
                let c: Nat satisfy { d * c = p.pow(k.suc) }
                p.pow(k.suc) = p * p.pow(k)
                d * c = p * p.pow(k)
                if d.gcd(p) = Nat.1 {
                    d.coprime(p)
                    d.divides(p * p.pow(k))
                    coprime_divides_of_divides_mul(d, p, p.pow(k))
                    d.divides(p.pow(k))
                    d.divides(p.pow(k)) implies exists(i: Nat) { d = p.pow(i) and i <= k }
                    let i: Nat satisfy { d = p.pow(i) and i <= k }
                    d = p.pow(i)
                    i <= k
                    k <= k.suc
                    lte_trans(i, k, k.suc)
                    i <= k.suc
                    d = p.pow(i) and i <= k.suc
                    exists(i0: Nat) { d = p.pow(i0) and i0 <= k.suc }
                } else {
                    d.gcd(p) != Nat.1
                    gcd_divides_right(d, p)
                    d.gcd(p).divides(p)
                    prime_divisor_is_one_or_self(p, d.gcd(p))
                    d.gcd(p) = Nat.1 or d.gcd(p) = p
                    d.gcd(p) = p
                    gcd_divides_left(d, p)
                    p.divides(d)
                    let d0: Nat satisfy { p * d0 = d }
                    d = p * d0
                    p * d0 * c = p * p.pow(k)
                    p * (d0 * c) = p * p.pow(k)
                    mul_cancel_left(p, d0 * c, p.pow(k))
                    d0 * c = p.pow(k)
                    d0.divides(p.pow(k))
                    d0.divides(p.pow(k)) implies exists(i: Nat) { d0 = p.pow(i) and i <= k }
                    let i: Nat satisfy { d0 = p.pow(i) and i <= k }
                    d0 = p.pow(i)
                    d = p * p.pow(i)
                    p.pow(i.suc) = p * p.pow(i)
                    p * p.pow(i) = p.pow(i.suc)
                    d = p.pow(i.suc)
                    i <= k
                    i.suc <= k.suc
                    d = p.pow(i.suc) and i.suc <= k.suc
                    exists(i0: Nat) { d = p.pow(i0) and i0 <= k.suc }
                }
                exists(i0: Nat) { d = p.pow(i0) and i0 <= k.suc }
            }
        }
    }
}

/// Every divisor of the prime power `p^k` is a power `p^i` with `i <= k`.
theorem prime_power_divisor(p: Nat, k: Nat) {
    p.is_prime implies forall(d: Nat) {
        d.divides(p.pow(k)) implies exists(i: Nat) { d = p.pow(i) and i <= k }
    }
} by {
    let f: Nat -> Bool = function(j: Nat) {
        p.is_prime implies forall(d: Nat) {
            d.divides(p.pow(j)) implies exists(i: Nat) { d = p.pow(i) and i <= j }
        }
    }
    prime_power_divisor_base_forall(p)
    if p.is_prime {
        forall(d: Nat) {
            d.divides(p.pow(Nat.0)) implies exists(i: Nat) { d = p.pow(i) and i <= Nat.0 }
        }
    }
    f(Nat.0)
    forall(j: Nat) {
        if f(j) {
            if p.is_prime {
                forall(d: Nat) {
                    d.divides(p.pow(j)) implies exists(i: Nat) { d = p.pow(i) and i <= j }
                }
                prime_power_divisor_step_forall(p, j)
                forall(d: Nat) {
                    d.divides(p.pow(j.suc)) implies exists(i: Nat) { d = p.pow(i) and i <= j.suc }
                }
            }
            f(j.suc)
        }
    }
    f(k)
}

/// A power `p^i` divides `p^k` whenever `i <= k`.
theorem prime_power_divides(p: Nat, i: Nat, k: Nat) {
    i <= k implies p.pow(i).divides(p.pow(k))
} by {
    if i <= k {
        let e: Nat satisfy { i + e = k }
        exp_add(p, i, e)
        p.pow(i + e) = p.pow(i) * p.pow(e)
        p.pow(i) * p.pow(e) = p.pow(i + e)
        p.pow(i + e) = p.pow(k)
        p.pow(i) * p.pow(e) = p.pow(k)
        exists(c: Nat) { p.pow(i) * c = p.pow(k) }
        p.pow(i).divides(p.pow(k))
    }
}

/// A power of a prime is positive.
theorem prime_power_pos(p: Nat, i: Nat) {
    p != Nat.0 implies Nat.0 < p.pow(i)
} by {
    if p != Nat.0 {
        exp_ne_zero(p, i)
        p.pow(i) != Nat.0
        pos_of_ne_zero(p.pow(i))
        Nat.0 < p.pow(i)
    }
}

/// Powers of a prime are monotone in the exponent.
theorem prime_power_le(p: Nat, i: Nat, k: Nat) {
    p != Nat.0 and i <= k implies p.pow(i) <= p.pow(k)
} by {
    if p != Nat.0 and i <= k {
        lte_imp_exp_lte(p, i, k)
        p.pow(i) <= p.pow(k)
    }
}

/// The map sending an exponent to the corresponding power of `p`.
define prime_power_fn(p: Nat) -> (Nat -> Nat) {
    function(i: Nat) { p.pow(i) }
}

/// The list `[p^0, ..., p^k]` of the first `k + 1` powers of `p`.
define prime_power_range(p: Nat, k: Nat) -> List[Nat] {
    map(k.suc.range, prime_power_fn(p))
}

/// The powers list has length `k + 1`.
theorem prime_power_range_length(p: Nat, k: Nat) {
    prime_power_range(p, k).length = k + Nat.1
} by {
    map_length[Nat, Nat](k.suc.range, prime_power_fn(p))
    prime_power_range(p, k).length = k.suc.range.length
    length_range(k.suc)
    k.suc.range.length = k.suc
    prime_power_range(p, k).length = k.suc
    k.suc = k + Nat.1
    prime_power_range(p, k).length = k + Nat.1
}

/// Distinct exponents give distinct powers of a prime, so the powers list is
/// unique.
theorem prime_power_range_unique(p: Nat, k: Nat) {
    p.is_prime implies prime_power_range(p, k).is_unique
} by {
    if p.is_prime {
        Nat.1 < p
        range_is_unique(k.suc)
        k.suc.range.is_unique
        forall(x: Nat, y: Nat) {
            if k.suc.range.contains(x) and k.suc.range.contains(y) and
                prime_power_fn(p)(x) = prime_power_fn(p)(y) {
                prime_power_fn(p)(x) = p.pow(x)
                prime_power_fn(p)(y) = p.pow(y)
                p.pow(x) = p.pow(y)
                lt_of_range_contains(k.suc, x)
                x < k.suc
                x <= k
                lt_of_range_contains(k.suc, y)
                y < k.suc
                y <= k
                p.pow(x) <= p.pow(y)
                exp_lte_imp_lte(p, x, y)
                x <= y
                p.pow(y) <= p.pow(x)
                exp_lte_imp_lte(p, y, x)
                y <= x
                lte_antisymm(x, y)
                x = y
            }
        }
        locally_injective_map_is_unique[Nat, Nat](k.suc.range, prime_power_fn(p))
        map(k.suc.range, prime_power_fn(p)).is_unique
        prime_power_range(p, k).is_unique
    }
}

/// Every power `p^i` with `i <= k` lies in the powers list.
theorem prime_power_range_contains(p: Nat, k: Nat, i: Nat) {
    i <= k implies prime_power_range(p, k).contains(p.pow(i))
} by {
    if i <= k {
        k <= k.suc
        lte_and_lt(i, k, k.suc)
        i < k.suc
        range_contains_of_lt(k.suc, i)
        k.suc.range.contains(i)
        map_contains_of_contains[Nat, Nat](k.suc.range, prime_power_fn(p), i)
        prime_power_range(p, k).contains(prime_power_fn(p)(i))
        prime_power_fn(p)(i) = p.pow(i)
        prime_power_range(p, k).contains(p.pow(i))
    }
}

/// Every power `p^i` with `i <= k` is a divisor of `p^k`.
theorem prime_power_in_divisor_list(p: Nat, k: Nat, i: Nat) {
    p.is_prime and i <= k implies divisor_list(p.pow(k)).contains(p.pow(i))
} by {
    if p.is_prime and i <= k {
        Nat.1 < p
        p != Nat.0
        prime_power_divides(p, i, k)
        p.pow(i).divides(p.pow(k))
        prime_power_pos(p, i)
        Nat.0 < p.pow(i)
        prime_power_le(p, i, k)
        p.pow(i) <= p.pow(k)
        divisors_up_to_complete(p.pow(k), p.pow(k), p.pow(i))
        divisors_up_to(p.pow(k), p.pow(k)).contains(p.pow(i))
        divisor_list(p.pow(k)) = divisors_up_to(p.pow(k), p.pow(k))
        divisor_list(p.pow(k)).contains(p.pow(i))
    }
}

/// Every member of the divisor list of `p^k` is a power of `p`.
theorem prime_power_divisor_in_range(p: Nat, k: Nat, d: Nat) {
    p.is_prime and divisor_list(p.pow(k)).contains(d)
        implies exists(i: Nat) { d = p.pow(i) and i <= k }
} by {
    if p.is_prime and divisor_list(p.pow(k)).contains(d) {
        divisor_list_contains_implies(p.pow(k), d)
        Nat.0 < d
        d.divides(p.pow(k))
        prime_power_divisor(p, k)
        forall(dd: Nat) {
            dd.divides(p.pow(k)) implies exists(ii: Nat) { dd = p.pow(ii) and ii <= k }
        }
        d.divides(p.pow(k)) implies exists(i: Nat) { d = p.pow(i) and i <= k }
        exists(i: Nat) { d = p.pow(i) and i <= k }
    }
}

/// The divisor list of `p^k` is contained in the powers list.
theorem divisor_list_contained_in_prime_power_range(p: Nat, k: Nat) {
    p.is_prime implies forall(x: Nat) {
        divisor_list(p.pow(k)).contains(x) implies prime_power_range(p, k).contains(x)
    }
} by {
    if p.is_prime {
        forall(x: Nat) {
            if divisor_list(p.pow(k)).contains(x) {
                prime_power_divisor_in_range(p, k, x)
                let i: Nat satisfy { x = p.pow(i) and i <= k }
                x = p.pow(i)
                i <= k
                prime_power_range_contains(p, k, i)
                prime_power_range(p, k).contains(p.pow(i))
                prime_power_range(p, k).contains(x)
            }
        }
    }
}

/// The powers list is contained in the divisor list of `p^k`.
theorem prime_power_range_contained_in_divisor_list(p: Nat, k: Nat) {
    p.is_prime implies forall(x: Nat) {
        prime_power_range(p, k).contains(x) implies divisor_list(p.pow(k)).contains(x)
    }
} by {
    if p.is_prime {
        forall(x: Nat) {
            if prime_power_range(p, k).contains(x) {
                prime_power_range(p, k) = map(k.suc.range, prime_power_fn(p))
                map_contains[Nat, Nat](k.suc.range, prime_power_fn(p), x)
                let i: Nat satisfy { k.suc.range.contains(i) and prime_power_fn(p)(i) = x }
                lt_of_range_contains(k.suc, i)
                i < k.suc
                i <= k
                prime_power_in_divisor_list(p, k, i)
                divisor_list(p.pow(k)).contains(p.pow(i))
                prime_power_fn(p)(i) = p.pow(i)
                prime_power_fn(p)(i) = x
                x = p.pow(i)
                divisor_list(p.pow(k)).contains(x)
            }
        }
    }
}

/// `tau(p^k) = k + 1` for a prime `p`.
///
/// The unique divisor list of `p^k` contains exactly the powers `p^0, ..., p^k`
/// (both containments above), so by `unique_is_smallest_containing_list` its
/// length is bounded above and below by the powers-list length `k + 1`.
theorem nat_tau_prime_pow(p: Nat, k: Nat) {
    p.is_prime implies nat_tau(p.pow(k)) = k + Nat.1
} by {
    if p.is_prime {
        divisor_list_contained_in_prime_power_range(p, k)
        unique_is_smallest_containing_list[Nat](divisor_list(p.pow(k)), prime_power_range(p, k))
        divisor_list(p.pow(k)).unique.length <= prime_power_range(p, k).length
        divisor_list_is_unique(p.pow(k))
        divisor_list(p.pow(k)).is_unique
        divisor_list(p.pow(k)).unique = divisor_list(p.pow(k))
        divisor_list(p.pow(k)).unique.length = divisor_list(p.pow(k)).length
        divisor_list(p.pow(k)).length <= prime_power_range(p, k).length
        prime_power_range_length(p, k)
        prime_power_range(p, k).length = k + Nat.1
        divisor_list(p.pow(k)).length <= k + Nat.1
        nat_tau(p.pow(k)) = divisor_list(p.pow(k)).length
        nat_tau(p.pow(k)) <= k + Nat.1

        prime_power_range_contained_in_divisor_list(p, k)
        unique_is_smallest_containing_list[Nat](prime_power_range(p, k), divisor_list(p.pow(k)))
        prime_power_range(p, k).unique.length <= divisor_list(p.pow(k)).length
        prime_power_range_unique(p, k)
        prime_power_range(p, k).is_unique
        prime_power_range(p, k).unique = prime_power_range(p, k)
        prime_power_range(p, k).unique.length = prime_power_range(p, k).length
        prime_power_range(p, k).length <= divisor_list(p.pow(k)).length
        prime_power_range_length(p, k)
        prime_power_range(p, k).length = k + Nat.1
        k + Nat.1 <= divisor_list(p.pow(k)).length
        k + Nat.1 <= nat_tau(p.pow(k))
        lte_antisymm(nat_tau(p.pow(k)), k + Nat.1)
        nat_tau(p.pow(k)) = k + Nat.1
    }
}

// ---------------------------------------------------------------------------
// Bounds on tau.
//
// The classical divisor-pairing bound is `tau(n) <= 2 sqrt(n)`: every divisor
// `d` of `n` pairs with the cofactor `n / d`, and each pair contains a member
// at most `sqrt(n)`, so the divisor list has at most `2 sqrt(n)` entries.  A
// sharp Nat version needs an integer square root for arbitrary `n` (the
// library's `isqrt` is only developed for primes), so we state the two
// Nat-friendly weakenings below: `tau(n) <= n + 1` and `tau(n) <= 2 n`, since
// `2 sqrt(n) <= n + 1 <= 2 n` for `n >= 1`.
// ---------------------------------------------------------------------------

/// A strictly positive divisor of `n` is at most `n`.
theorem positive_divisor_lte_self(n: Nat, d: Nat) {
    Nat.0 < d and d.divides(n) and n != Nat.0 implies d <= n
} by {
    if Nat.0 < d and d.divides(n) and n != Nat.0 {
        divides_lte(d, n)
        n = Nat.0 or d <= n
        d <= n
    }
}

/// The number of divisors of `n` is at most `n + 1`.
///
/// The divisor list of `n` is unique (divisor_list_is_unique) and every entry
/// is a positive divisor of `n`, hence lies in `{1, ..., n}`, a subset of the
/// range below `n + 1`; a unique list contained in a list of length `n + 1`
/// has length at most `n + 1`.
theorem nat_tau_le_nat_add_one(n: Nat) {
    nat_tau(n) <= n + Nat.1
} by {
    if n = Nat.0 {
        nat_tau_zero
        nat_tau(Nat.0) = Nat.0
        Nat.0 <= Nat.0 + Nat.1
        nat_tau(n) <= n + Nat.1
    } else {
        n != Nat.0
        forall(d: Nat) {
            if divisor_list(n).contains(d) {
                divisor_list_contains_implies(n, d)
                Nat.0 < d
                d.divides(n)
                positive_divisor_lte_self(n, d)
                d <= n
                n < n + Nat.1
                lte_and_lt(d, n, n + Nat.1)
                d < n + Nat.1
                range_contains_of_lt(n + Nat.1, d)
                (n + Nat.1).range.contains(d)
            }
        }
        unique_is_smallest_containing_list[Nat](divisor_list(n), (n + Nat.1).range)
        divisor_list(n).unique.length <= (n + Nat.1).range.length
        divisor_list_is_unique(n)
        divisor_list(n).is_unique
        divisor_list(n).unique = divisor_list(n)
        divisor_list(n).unique.length = divisor_list(n).length
        length_range(n + Nat.1)
        (n + Nat.1).range.length = n + Nat.1
        divisor_list(n).length <= n + Nat.1
        nat_tau(n) = divisor_list(n).length
        nat_tau(n) <= n + Nat.1
    }
}

/// A Nat-friendly form of the classical bound `tau(n) <= 2 sqrt(n)`:
/// `tau(n) <= 2 n`.
theorem nat_tau_le_two_mul(n: Nat) {
    nat_tau(n) <= Nat.2 * n
} by {
    if n = Nat.0 {
        nat_tau_zero
        nat_tau(Nat.0) = Nat.0
        nat_tau(n) = nat_tau(Nat.0)
        nat_tau(n) = Nat.0
        Nat.2 * Nat.0 = Nat.0
        Nat.2 * n = Nat.2 * Nat.0
        Nat.2 * n = Nat.0
        Nat.0 <= Nat.0
        nat_tau(n) <= Nat.2 * n
    } else {
        n != Nat.0
        pos_of_ne_zero(n)
        Nat.0 < n
        Nat.1 <= n
        nat_tau_le_nat_add_one(n)
        nat_tau(n) <= n + Nat.1
        Nat.1 <= n
        n + Nat.1 <= n + n
        Nat.2 * n = n + n
        n + Nat.1 <= Nat.2 * n
        lte_trans(nat_tau(n), n + Nat.1, Nat.2 * n)
        nat_tau(n) <= Nat.2 * n
    }
}

// ---------------------------------------------------------------------------
// Characterising primes by their divisor count.
// ---------------------------------------------------------------------------

/// A list of length zero is empty.
theorem length_zero_imp_nil[T](list: List[T]) {
    list.length = Nat.0 implies list = List.nil[T]
} by {
    let p: List[T] -> Bool = function(l: List[T]) {
        l.length = Nat.0 implies l = List.nil[T]
    }
    List.nil[T].length = Nat.0
    p(List.nil[T])
    forall(head: T, tail: List[T]) {
        if p(tail) {
            if List.cons(head, tail).length = Nat.0 {
                List.cons(head, tail).length = tail.length.suc
                tail.length.suc = Nat.0
                false
            }
            p(List.cons(head, tail))
        }
    }
    p(list)
}

/// A list of length one is a singleton.
theorem length_one_cons[T](list: List[T]) {
    list.length = Nat.1 implies exists(a: T) { list = List.cons(a, List.nil[T]) }
} by {
    let p: List[T] -> Bool = function(l: List[T]) {
        l.length = Nat.1 implies exists(a: T) { l = List.cons(a, List.nil[T]) }
    }
    if List.nil[T].length = Nat.1 {
        List.nil[T].length = Nat.0
        Nat.0 = Nat.1
        false
    }
    p(List.nil[T])
    forall(head: T, tail: List[T]) {
        if p(tail) {
            if List.cons(head, tail).length = Nat.1 {
                List.cons(head, tail).length = tail.length.suc
                tail.length.suc = Nat.1
                tail.length.suc = Nat.0.suc
                tail.length = Nat.0
                length_zero_imp_nil[T](tail)
                tail = List.nil[T]
                List.cons(head, tail) = List.cons(head, List.nil[T])
                exists(a: T) { List.cons(head, tail) = List.cons(a, List.nil[T]) }
            }
            p(List.cons(head, tail))
        }
    }
    p(list)
}

/// A list of length two is a pair.
theorem length_two_cons[T](list: List[T]) {
    list.length = Nat.2 implies exists(a: T, b: T) {
        list = List.cons(a, List.cons(b, List.nil[T]))
    }
} by {
    let p: List[T] -> Bool = function(l: List[T]) {
        l.length = Nat.2 implies exists(a: T, b: T) {
            l = List.cons(a, List.cons(b, List.nil[T]))
        }
    }
    if List.nil[T].length = Nat.2 {
        List.nil[T].length = Nat.0
        Nat.0 = Nat.2
        false
    }
    p(List.nil[T])
    forall(head: T, tail: List[T]) {
        if p(tail) {
            if List.cons(head, tail).length = Nat.2 {
                List.cons(head, tail).length = tail.length.suc
                tail.length.suc = Nat.2
                tail.length.suc = Nat.1.suc
                tail.length = Nat.1
                length_one_cons[T](tail)
                let b: T satisfy { tail = List.cons(b, List.nil[T]) }
                tail = List.cons(b, List.nil[T])
                List.cons(head, tail) = List.cons(head, List.cons(b, List.nil[T]))
                exists(a: T, bb: T) {
                    List.cons(head, tail) = List.cons(a, List.cons(bb, List.nil[T]))
                }
            }
            p(List.cons(head, tail))
        }
    }
    p(list)
}

/// Membership in a two-element list forces equality with one of the entries.
theorem two_elem_contains_imp_eq[T](x: T, y: T, a: T) {
    List.cons(x, List.cons(y, List.nil[T])).contains(a) implies (a = x or a = y)
} by {
    if List.cons(x, List.cons(y, List.nil[T])).contains(a) {
        if a = x {
            a = x or a = y
        } else {
            a != x
            List.cons(y, List.nil[T]).contains(a)
            singleton_contains_imp_eq[T](y, a)
            a = y
            a = x or a = y
        }
    }
}

/// A number with exactly two positive divisors is prime.
///
/// The divisor list of `n` has length two and is unique, so it is `[x, y]`
/// for two values; since `1` and `n` both occur and are distinct, the two
/// entries are exactly `1` and `n`, leaving no room for any proper divisor
/// `d` with `1 < d < n`.
theorem nat_tau_two_imp_prime(n: Nat) {
    nat_tau(n) = Nat.2 implies n.is_prime
} by {
    if nat_tau(n) = Nat.2 {
        if n = Nat.0 {
            nat_tau_zero
            nat_tau(Nat.0) = Nat.0
            Nat.0 = Nat.2
            false
        }
        n != Nat.0
        if n = Nat.1 {
            nat_tau_one
            nat_tau(Nat.1) = Nat.1
            Nat.1 = Nat.2
            false
        }
        n != Nat.1
        pos_of_ne_zero(n)
        Nat.0 < n
        Nat.1 <= n
        lte_neq_imp_lt(Nat.1, n)
        Nat.1 < n
        forall(d: Nat) {
            if Nat.1 < d and d < n {
                if d.divides(n) {
                    Nat.1 < d
                    Nat.0 < d
                    divisor_list_contains_of(n, d)
                    divisor_list(n).contains(d)
                    one_divides_nat(n)
                    Nat.1.divides(n)
                    divisor_list_contains_of(n, Nat.1)
                    divisor_list(n).contains(Nat.1)
                    divides_self(n)
                    n.divides(n)
                    divisor_list_contains_of(n, n)
                    divisor_list(n).contains(n)
                    nat_tau(n) = divisor_list(n).length
                    divisor_list(n).length = Nat.2
                    length_two_cons[Nat](divisor_list(n))
                    let (x: Nat, y: Nat) satisfy {
                        divisor_list(n) = List.cons(x, List.cons(y, List.nil[Nat]))
                    }
                    divisor_list(n) = List.cons(x, List.cons(y, List.nil[Nat]))
                    List.cons(x, List.cons(y, List.nil[Nat])).contains(Nat.1)
                    List.cons(x, List.cons(y, List.nil[Nat])).contains(n)
                    List.cons(x, List.cons(y, List.nil[Nat])).contains(d)
                    two_elem_contains_imp_eq[Nat](x, y, Nat.1)
                    Nat.1 = x or Nat.1 = y
                    two_elem_contains_imp_eq[Nat](x, y, n)
                    n = x or n = y
                    two_elem_contains_imp_eq[Nat](x, y, d)
                    d = x or d = y
                    Nat.1 != n
                    Nat.1 != d
                    d != n
                    if Nat.1 = x {
                        if n = x {
                            n = Nat.1
                            false
                        } else {
                            n = y
                            if d = x {
                                d = Nat.1
                                false
                            } else {
                                d = y
                                d = n
                                false
                            }
                        }
                    } else {
                        Nat.1 = y
                        if n = x {
                            if d = x {
                                d = n
                                false
                            } else {
                                d = y
                                d = Nat.1
                                false
                            }
                        } else {
                            n = y
                            n = Nat.1
                            false
                        }
                    }
                    false
                }
                not d.divides(n)
            }
        }
        no_proper_divisor_imp_prime(n)
        n.is_prime
    }
}

/// `tau(n) = 2` exactly when `n` is prime.
theorem nat_tau_two_iff_prime(n: Nat) {
    (nat_tau(n) = Nat.2) = n.is_prime
} by {
    if nat_tau(n) = Nat.2 {
        nat_tau_two_imp_prime(n)
        n.is_prime
    }
    if n.is_prime {
        nat_tau_prime(n)
        nat_tau(n) = Nat.2
    }
}
