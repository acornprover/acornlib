/// The unit-distance count of the four-dimensional grid.
///
/// This file formalizes the classical lower-bound side around the 2026
/// disproof of the Erdős unit-distance conjecture: the n by n by n by n
/// lattice grid in R⁴.  A grid point has exactly eight neighbours at
/// distance one (one in each of the four coordinate directions, in each of
/// the two signs), so the grid of n⁴ points carries exactly
/// 4·n³·(n − 1) unit distances — the analogue of the two-dimensional
/// grid count 2·n·(n − 1).
///
/// The counting uses the quaternion presentation of R⁴ from
/// `number_theory/quaternion_unit_distance.ac`: a point is a pair of
/// complex numbers, the squared distance is the norm of the difference, and
/// `nu` counts unit pairs.  This file proves the coordinate bounds behind
/// the characterization of the unit sphere in the four-dimensional integer
/// lattice: each coordinate of a natural solution of a² + b² + c² + d² = 1
/// is at most one (`from_nat_sq_le_one_imp_le_one`), hence zero or one
/// (`nat_le_one_zero_or_one`), and any two non-zero coordinates make the
/// sum of squares exceed one (`four_sq_two_ones_gt_one` and its five
/// permutations, covering all pairs of coordinates).  The full
/// characterization — exactly one coordinate equals one (`four_sq_eq_one`)
/// — and the grid count statements that follow from it are recorded as
/// commented theorems at the end of this file: the case analysis is
/// mechanical but long, and the count is a direct enumeration.
from real import Real, square_nonneg, mul_lt_mul_of_pos_right, from_nat_real_pos_of_ne_zero, add_lte_add
from nat import Nat, from_nat, from_nat_add, from_nat_zero, from_nat_one,
    lt_diff, add_comm, lte_antisymm, lt_imp_lte_suc, add_sub,
    sub_zero, add_imp_sub, lt_or_lte, sub_lt, pos_of_ne_zero
from order import lt_imp_lte, lt_not_ref, lt_trans, lt_of_lt_of_lte, lt_of_lte_of_lt, lte_trans,
    lt_of_lt_of_eq
from pair import Pair, pair_new_first, pair_new_second, pair_ext
from finite_set import FiniteSet, fs_image, finite_set_image_contains_eq, finite_set_ext_contains,
    finite_set_image_cardinality_is_of_injective
from data.basic.functions import is_injective_fn
from data.finite.finite_set_card import fs_card, fs_card_eq_of_cardinality_is, fs_card_cardinality_is
from data.finite.finite_set_product import finite_set_product, finite_set_product_contains_eq,
    finite_set_product_contains_pair
from data.finite.finite_set_product_card import fs_card_product
from data.nat.nat_range_set import range_set, range_set_contains_eq, range_set_card
from complex import Complex, abs_squared_eq
from number_theory.four_squares_identity import norm2
from number_theory.quaternion_unit_distance import Quat, quat_norm, quat_sub, quat_add, nu,
    ordered_unit_pairs, ordered_unit_pair, quaternion_unit_distance, quat_dist_sq,
    quat_dist_sq_add_right

numerals Real
numerals Nat

// ============================================================================
// The unit sphere of the four-dimensional integer lattice
// ============================================================================

/// Zero is strictly less than one in the reals.
theorem real_zero_lt_one {
    Real.0 < Real.1
} by {
    from_nat_real_pos_of_ne_zero(Nat.1)
    Nat.1 != Nat.0
    from_nat[Real](Nat.1) > Real.0
    from_nat_one[Real]
    from_nat[Real](Nat.1) = Real.1
    Real.1 > Real.0
}

/// Zero is at most one in the reals.
theorem real_zero_lte_one {
    Real.0 <= Real.1
} by {
    real_zero_lt_one
    Real.0 < Real.1
    lt_imp_lte(Real.0, Real.1)
    Real.0 <= Real.1
}

/// The image of a natural in the reals is nonnegative.
theorem from_nat_real_nonneg(k: Nat) {
    from_nat[Real](k) >= Real.0
} by {
    define p(m: Nat) -> Bool {
        from_nat[Real](m) >= Real.0
    }
    from_nat_zero[Real]
    from_nat[Real](Nat.0) = Real.0
    from_nat[Real](Nat.0) >= Real.0
    p(Nat.0)
    forall(m: Nat) {
        if p(m) {
            from_nat[Real](m) >= Real.0
            from_nat[Real](m.suc) = from_nat[Real](m) + Real.1
            real_zero_lte_one
            Real.0 <= Real.1
            from_nat[Real](m) + Real.0 <= from_nat[Real](m) + Real.1
            from_nat[Real](m.suc) >= Real.0
            p(m.suc)
        }
        p(m) implies p(m.suc)
    }
    p(Nat.0) and forall(m: Nat) { p(m) implies p(m.suc) }
    Nat.induction(p)
    p(k)
    from_nat[Real](k) >= Real.0
}

/// A square of a real is at most the sum of two squares.
theorem sq_le_sum_sq(a: Real, b: Real) {
    a * a <= a * a + b * b
} by {
    square_nonneg(b)
    Real.0 <= b * b
    a * a + Real.0 <= a * a + b * b
    a * a <= a * a + b * b
}

/// A sum of squares is nonnegative.
theorem two_sq_nonneg(a: Real, b: Real) {
    a * a + b * b >= Real.0
} by {
    square_nonneg(a)
    Real.0 <= a * a
    square_nonneg(b)
    Real.0 <= b * b
    add_lte_add(Real.0, a * a, Real.0, b * b)
    Real.0 + Real.0 <= a * a + b * b
    Real.0 + Real.0 = Real.0
    Real.0 <= a * a + b * b
}

/// Adding a nonnegative amount does not decrease: x <= x + y when y >= 0.
theorem le_add_nonneg(x: Real, y: Real) {
    y >= Real.0 implies x <= x + y
} by {
    if y >= Real.0 {
        add_lte_add(x, x, Real.0, y)
        x + Real.0 <= x + y
        x + Real.0 = x
        x <= x + y
    }
}

/// Rewriting the right side of a non-strict inequality by an equality.
theorem le_eq_rewrite(x: Real, y: Real, z: Real) {
    x <= y and y = z implies x <= z
} by {
}
/// A square of a real is at most the sum of four squares.
theorem sq_le_sum4(a: Real, b: Real, c: Real, d: Real) {
    a * a <= a * a + b * b + c * c + d * d
} by {
    square_nonneg(b)
    Real.0 <= b * b
    le_add_nonneg(a * a, b * b)
    a * a <= a * a + b * b
    two_sq_nonneg(c, d)
    c * c + d * d >= Real.0
    le_add_nonneg(a * a + b * b, c * c + d * d)
    a * a + b * b <= (a * a + b * b) + (c * c + d * d)
    (a * a + b * b) + (c * c + d * d) = a * a + b * b + c * c + d * d
    le_eq_rewrite(a * a + b * b, (a * a + b * b) + (c * c + d * d), a * a + b * b + c * c + d * d)
    a * a + b * b <= a * a + b * b + c * c + d * d
    lte_trans(a * a, a * a + b * b, a * a + b * b + c * c + d * d)
    a * a <= a * a + b * b
    a * a + b * b <= a * a + b * b + c * c + d * d
    a * a <= a * a + b * b + c * c + d * d
}


/// Adding a positive real makes things strictly larger.
theorem real_add_pos_lt(x: Real, y: Real) {
    y > Real.0 implies x < x + y
} by {
    if y > Real.0 {
        Real.0 < y
        Real.0 + x < y + x
        x < x + y
    }
}

/// `from_nat` is strictly monotone on the reals.
theorem from_nat_lt_strict(m: Nat, n: Nat) {
    m < n implies from_nat[Real](m) < from_nat[Real](n)
} by {
    if m < n {
        lt_diff(m, n)
        exists(d: Nat) { m + d = n and d != Nat.0 }
        let d: Nat satisfy { m + d = n and d != Nat.0 }
        from_nat_add[Real](m, d)
        from_nat[Real](m + d) = from_nat[Real](m) + from_nat[Real](d)
        from_nat[Real](d) > Real.0
        real_add_pos_lt(from_nat[Real](m), from_nat[Real](d))
        from_nat[Real](m) < from_nat[Real](m) + from_nat[Real](d)
        from_nat[Real](m) + from_nat[Real](d) = from_nat[Real](n)
        from_nat[Real](m) < from_nat[Real](n)
    }
}

/// `from_nat` reflects the non-strict order.
theorem from_nat_le_reflect(m: Nat, n: Nat) {
    from_nat[Real](m) <= from_nat[Real](n) implies m <= n
} by {
    if from_nat[Real](m) <= from_nat[Real](n) {
        lt_or_lte(n, m)
        if n < m {
            from_nat_lt_strict(n, m)
            from_nat[Real](n) < from_nat[Real](m)
            false
        }
        m <= n
    }
}

/// A nonnegative real whose square is at most one is at most one.
theorem real_sq_le_one_imp_le_one(x: Real) {
    x >= Real.0 and x * x <= Real.1 implies x <= Real.1
} by {
    if x >= Real.0 and x * x <= Real.1 {
        if Real.1 < x {
            real_zero_lt_one
            Real.0 < Real.1
            lt_of_lt_of_lte(Real.0, Real.1, x)
            Real.0 < x
            mul_lt_mul_of_pos_right(Real.1, x, x)
            Real.1 * x < x * x
            Real.1 * x = x
            x < x * x
            x * x <= Real.1
            lt_of_lte_of_lt(x * x, Real.1, x)
            x * x < x
            lt_trans(x, x * x, x)
            x < x
            lt_not_ref(x)
            false
        }
        x <= Real.1
    }
}

/// The square of a nonnegative image of a natural is at most one only for
/// naturals at most one.
theorem from_nat_sq_le_one_imp_le_one(k: Nat) {
    from_nat[Real](k) * from_nat[Real](k) <= Real.1 implies k <= Nat.1
} by {
    if from_nat[Real](k) * from_nat[Real](k) <= Real.1 {
        from_nat_real_nonneg(k)
        from_nat[Real](k) >= Real.0
        real_sq_le_one_imp_le_one(from_nat[Real](k))
        from_nat[Real](k) <= Real.1
        from_nat_one[Real]
        from_nat[Real](Nat.1) = Real.1
        from_nat[Real](k) <= from_nat[Real](Nat.1)
        from_nat_le_reflect(k, Nat.1)
        k <= Nat.1
    }
}

/// A natural at most one is zero or one.
theorem nat_le_one_zero_or_one(k: Nat) {
    k <= Nat.1 implies (k = Nat.0 or k = Nat.1)
} by {
    if k <= Nat.1 {
        if k = Nat.0 {
            k = Nat.0 or k = Nat.1
        }
        if k != Nat.0 {
            Nat.0 < k
            lt_imp_lte_suc(Nat.0, k)
            Nat.1 <= k
            lte_antisymm(k, Nat.1)
            k = Nat.1
            k = Nat.0 or k = Nat.1
        }
        k = Nat.0 or k = Nat.1
    }
}

/// Two ones among the four sum to more than one.
theorem two_ones_sum_gt_one(a: Real, b: Real) {
    a = Real.1 and b = Real.1 implies
        a * a + b * b > Real.1
} by {
    if a = Real.1 and b = Real.1 {
        a * a = Real.1
        b * b = Real.1
        a * a + b * b = Real.1 + Real.1
        Real.0 < Real.1
        real_add_pos_lt(Real.1, Real.1)
        Real.1 < Real.1 + Real.1
        a * a + b * b > Real.1
    }
}

/// A sum of two squares is at most a sum of four squares containing it.
theorem sq_le_sum4_swap(a: Real, b: Real, c: Real, d: Real) {
    a * a + b * b <= a * a + b * b + c * c + d * d
} by {
    two_sq_nonneg(c, d)
    c * c + d * d >= Real.0
    le_add_nonneg(a * a + b * b, c * c + d * d)
    a * a + b * b <= (a * a + b * b) + (c * c + d * d)
    (a * a + b * b) + (c * c + d * d) = a * a + b * b + c * c + d * d
    a * a + b * b <= a * a + b * b + c * c + d * d
}

/// If two of the four naturals are one, the sum of their squares is more
/// than one.  The two one-squares sit at the positions of the first two
/// arguments.
theorem four_sq_two_ones_gt_one(a: Nat, b: Nat, c: Nat, d: Nat) {
    a = Nat.1 and b = Nat.1 implies
    from_nat[Real](a) * from_nat[Real](a) +
    from_nat[Real](b) * from_nat[Real](b) +
    from_nat[Real](c) * from_nat[Real](c) +
    from_nat[Real](d) * from_nat[Real](d) > Real.1
} by {
    if a = Nat.1 and b = Nat.1 {
        from_nat_one[Real]
        from_nat[Real](Nat.1) = Real.1
        from_nat[Real](a) = Real.1
        from_nat[Real](b) = Real.1
        from_nat[Real](a) * from_nat[Real](a) = Real.1
        from_nat[Real](b) * from_nat[Real](b) = Real.1
        from_nat[Real](a) * from_nat[Real](a) +
            from_nat[Real](b) * from_nat[Real](b) +
            from_nat[Real](c) * from_nat[Real](c) +
            from_nat[Real](d) * from_nat[Real](d) =
            Real.1 + Real.1 +
            from_nat[Real](c) * from_nat[Real](c) +
            from_nat[Real](d) * from_nat[Real](d)
        two_sq_nonneg(from_nat[Real](c), from_nat[Real](d))
        from_nat[Real](c) * from_nat[Real](c) +
            from_nat[Real](d) * from_nat[Real](d) >= Real.0
        le_add_nonneg(Real.1 + Real.1,
            from_nat[Real](c) * from_nat[Real](c) +
            from_nat[Real](d) * from_nat[Real](d))
        Real.1 + Real.1 <= Real.1 + Real.1 +
            from_nat[Real](c) * from_nat[Real](c) +
            from_nat[Real](d) * from_nat[Real](d)
        le_eq_rewrite(Real.1 + Real.1,
            Real.1 + Real.1 + from_nat[Real](c) * from_nat[Real](c) +
            from_nat[Real](d) * from_nat[Real](d),
            from_nat[Real](a) * from_nat[Real](a) +
            from_nat[Real](b) * from_nat[Real](b) +
            from_nat[Real](c) * from_nat[Real](c) +
            from_nat[Real](d) * from_nat[Real](d))
        Real.1 + Real.1 <= from_nat[Real](a) * from_nat[Real](a) +
            from_nat[Real](b) * from_nat[Real](b) +
            from_nat[Real](c) * from_nat[Real](c) +
            from_nat[Real](d) * from_nat[Real](d)
        real_add_pos_lt(Real.1, Real.1)
        Real.1 < Real.1 + Real.1
        lt_imp_lte(Real.1, Real.1 + Real.1)
        Real.1 <= Real.1 + Real.1
        lt_of_lt_of_lte(Real.1, Real.1 + Real.1,
            from_nat[Real](a) * from_nat[Real](a) +
            from_nat[Real](b) * from_nat[Real](b) +
            from_nat[Real](c) * from_nat[Real](c) +
            from_nat[Real](d) * from_nat[Real](d))
        Real.1 < from_nat[Real](a) * from_nat[Real](a) +
            from_nat[Real](b) * from_nat[Real](b) +
            from_nat[Real](c) * from_nat[Real](c) +
            from_nat[Real](d) * from_nat[Real](d)
        from_nat[Real](a) * from_nat[Real](a) +
            from_nat[Real](b) * from_nat[Real](b) +
            from_nat[Real](c) * from_nat[Real](c) +
            from_nat[Real](d) * from_nat[Real](d) > Real.1
    }
}

/// If the first and third of the four naturals are one and the sum of their
/// squares is one, then false.
theorem four_sq_two_ones_gt_one_13(a: Nat, b: Nat, c: Nat, d: Nat) {
    a = Nat.1 and c = Nat.1 implies
    from_nat[Real](a) * from_nat[Real](a) +
    from_nat[Real](b) * from_nat[Real](b) +
    from_nat[Real](c) * from_nat[Real](c) +
    from_nat[Real](d) * from_nat[Real](d) > Real.1
} by {
    if a = Nat.1 and c = Nat.1 {
        from_nat_one[Real]
        from_nat[Real](Nat.1) = Real.1
        from_nat[Real](a) = Real.1
        from_nat[Real](c) = Real.1
        from_nat[Real](a) * from_nat[Real](a) = Real.1
        from_nat[Real](c) * from_nat[Real](c) = Real.1
        from_nat[Real](a) * from_nat[Real](a) +
            from_nat[Real](b) * from_nat[Real](b) +
            from_nat[Real](c) * from_nat[Real](c) +
            from_nat[Real](d) * from_nat[Real](d) =
            Real.1 + from_nat[Real](b) * from_nat[Real](b) + Real.1 +
            from_nat[Real](d) * from_nat[Real](d)
        square_nonneg(from_nat[Real](b))
        Real.0 <= from_nat[Real](b) * from_nat[Real](b)
        le_add_nonneg(Real.1, from_nat[Real](b) * from_nat[Real](b))
        Real.1 <= Real.1 + from_nat[Real](b) * from_nat[Real](b)
        square_nonneg(from_nat[Real](d))
        Real.0 <= from_nat[Real](d) * from_nat[Real](d)
        le_add_nonneg(Real.1, from_nat[Real](d) * from_nat[Real](d))
        Real.1 <= Real.1 + from_nat[Real](d) * from_nat[Real](d)
        add_lte_add(Real.1, Real.1 + from_nat[Real](b) * from_nat[Real](b), Real.1,
            Real.1 + from_nat[Real](d) * from_nat[Real](d))
        Real.1 + Real.1 <= Real.1 + from_nat[Real](b) * from_nat[Real](b) + Real.1 +
            from_nat[Real](d) * from_nat[Real](d)
        le_eq_rewrite(Real.1 + Real.1,
            Real.1 + from_nat[Real](b) * from_nat[Real](b) + Real.1 +
            from_nat[Real](d) * from_nat[Real](d),
            from_nat[Real](a) * from_nat[Real](a) +
            from_nat[Real](b) * from_nat[Real](b) +
            from_nat[Real](c) * from_nat[Real](c) +
            from_nat[Real](d) * from_nat[Real](d))
        Real.1 + Real.1 <= from_nat[Real](a) * from_nat[Real](a) +
            from_nat[Real](b) * from_nat[Real](b) +
            from_nat[Real](c) * from_nat[Real](c) +
            from_nat[Real](d) * from_nat[Real](d)
        real_add_pos_lt(Real.1, Real.1)
        Real.1 < Real.1 + Real.1
        lt_imp_lte(Real.1, Real.1 + Real.1)
        Real.1 <= Real.1 + Real.1
        lt_of_lt_of_lte(Real.1, Real.1 + Real.1,
            from_nat[Real](a) * from_nat[Real](a) +
            from_nat[Real](b) * from_nat[Real](b) +
            from_nat[Real](c) * from_nat[Real](c) +
            from_nat[Real](d) * from_nat[Real](d))
        Real.1 < from_nat[Real](a) * from_nat[Real](a) +
            from_nat[Real](b) * from_nat[Real](b) +
            from_nat[Real](c) * from_nat[Real](c) +
            from_nat[Real](d) * from_nat[Real](d)
        from_nat[Real](a) * from_nat[Real](a) +
            from_nat[Real](b) * from_nat[Real](b) +
            from_nat[Real](c) * from_nat[Real](c) +
            from_nat[Real](d) * from_nat[Real](d) > Real.1
    }
}

/// If the first and fourth of the four naturals are one and the sum of their
/// squares is one, then false.
theorem four_sq_two_ones_gt_one_14(a: Nat, b: Nat, c: Nat, d: Nat) {
    a = Nat.1 and d = Nat.1 implies
    from_nat[Real](a) * from_nat[Real](a) +
    from_nat[Real](b) * from_nat[Real](b) +
    from_nat[Real](c) * from_nat[Real](c) +
    from_nat[Real](d) * from_nat[Real](d) > Real.1
} by {
    if a = Nat.1 and d = Nat.1 {
        from_nat_one[Real]
        from_nat[Real](Nat.1) = Real.1
        from_nat[Real](a) = Real.1
        from_nat[Real](d) = Real.1
        from_nat[Real](a) * from_nat[Real](a) = Real.1
        from_nat[Real](d) * from_nat[Real](d) = Real.1
        from_nat[Real](a) * from_nat[Real](a) +
            from_nat[Real](b) * from_nat[Real](b) +
            from_nat[Real](c) * from_nat[Real](c) +
            from_nat[Real](d) * from_nat[Real](d) =
            Real.1 + from_nat[Real](b) * from_nat[Real](b) +
            from_nat[Real](c) * from_nat[Real](c) + Real.1
        square_nonneg(from_nat[Real](b))
        Real.0 <= from_nat[Real](b) * from_nat[Real](b)
        le_add_nonneg(Real.1, from_nat[Real](b) * from_nat[Real](b))
        Real.1 <= Real.1 + from_nat[Real](b) * from_nat[Real](b)
        square_nonneg(from_nat[Real](c))
        Real.0 <= from_nat[Real](c) * from_nat[Real](c)
        le_add_nonneg(Real.1, from_nat[Real](c) * from_nat[Real](c))
        Real.1 <= Real.1 + from_nat[Real](c) * from_nat[Real](c)
        Real.1 + from_nat[Real](c) * from_nat[Real](c) =
            from_nat[Real](c) * from_nat[Real](c) + Real.1
        le_eq_rewrite(Real.1, Real.1 + from_nat[Real](c) * from_nat[Real](c),
            from_nat[Real](c) * from_nat[Real](c) + Real.1)
        Real.1 <= from_nat[Real](c) * from_nat[Real](c) + Real.1
        add_lte_add(Real.1, Real.1 + from_nat[Real](b) * from_nat[Real](b), Real.1,
            from_nat[Real](c) * from_nat[Real](c) + Real.1)
        Real.1 + Real.1 <= Real.1 + from_nat[Real](b) * from_nat[Real](b) +
            from_nat[Real](c) * from_nat[Real](c) + Real.1
        le_eq_rewrite(Real.1 + Real.1,
            Real.1 + from_nat[Real](b) * from_nat[Real](b) +
            from_nat[Real](c) * from_nat[Real](c) + Real.1,
            from_nat[Real](a) * from_nat[Real](a) +
            from_nat[Real](b) * from_nat[Real](b) +
            from_nat[Real](c) * from_nat[Real](c) +
            from_nat[Real](d) * from_nat[Real](d))
        Real.1 + Real.1 <= from_nat[Real](a) * from_nat[Real](a) +
            from_nat[Real](b) * from_nat[Real](b) +
            from_nat[Real](c) * from_nat[Real](c) +
            from_nat[Real](d) * from_nat[Real](d)
        real_add_pos_lt(Real.1, Real.1)
        Real.1 < Real.1 + Real.1
        lt_imp_lte(Real.1, Real.1 + Real.1)
        Real.1 <= Real.1 + Real.1
        lt_of_lt_of_lte(Real.1, Real.1 + Real.1,
            from_nat[Real](a) * from_nat[Real](a) +
            from_nat[Real](b) * from_nat[Real](b) +
            from_nat[Real](c) * from_nat[Real](c) +
            from_nat[Real](d) * from_nat[Real](d))
        Real.1 < from_nat[Real](a) * from_nat[Real](a) +
            from_nat[Real](b) * from_nat[Real](b) +
            from_nat[Real](c) * from_nat[Real](c) +
            from_nat[Real](d) * from_nat[Real](d)
        from_nat[Real](a) * from_nat[Real](a) +
            from_nat[Real](b) * from_nat[Real](b) +
            from_nat[Real](c) * from_nat[Real](c) +
            from_nat[Real](d) * from_nat[Real](d) > Real.1
    }
}

/// If the second and third of the four naturals are one and the sum of
/// their squares is one, then false.
theorem four_sq_two_ones_gt_one_23(a: Nat, b: Nat, c: Nat, d: Nat) {
    b = Nat.1 and c = Nat.1 implies
    from_nat[Real](a) * from_nat[Real](a) +
    from_nat[Real](b) * from_nat[Real](b) +
    from_nat[Real](c) * from_nat[Real](c) +
    from_nat[Real](d) * from_nat[Real](d) > Real.1
} by {
    if b = Nat.1 and c = Nat.1 {
        from_nat_one[Real]
        from_nat[Real](Nat.1) = Real.1
        from_nat[Real](b) = Real.1
        from_nat[Real](c) = Real.1
        from_nat[Real](b) * from_nat[Real](b) = Real.1
        from_nat[Real](c) * from_nat[Real](c) = Real.1
        from_nat[Real](a) * from_nat[Real](a) +
            from_nat[Real](b) * from_nat[Real](b) +
            from_nat[Real](c) * from_nat[Real](c) +
            from_nat[Real](d) * from_nat[Real](d) =
            from_nat[Real](a) * from_nat[Real](a) + Real.1 + Real.1 +
            from_nat[Real](d) * from_nat[Real](d)
        square_nonneg(from_nat[Real](a))
        Real.0 <= from_nat[Real](a) * from_nat[Real](a)
        add_lte_add(Real.1, Real.1, Real.0, from_nat[Real](a) * from_nat[Real](a))
        Real.1 + Real.0 <= Real.1 + from_nat[Real](a) * from_nat[Real](a)
        Real.1 <= Real.1 + from_nat[Real](a) * from_nat[Real](a)
        Real.1 + from_nat[Real](a) * from_nat[Real](a) =
            from_nat[Real](a) * from_nat[Real](a) + Real.1
        le_eq_rewrite(Real.1, Real.1 + from_nat[Real](a) * from_nat[Real](a),
            from_nat[Real](a) * from_nat[Real](a) + Real.1)
        Real.1 <= from_nat[Real](a) * from_nat[Real](a) + Real.1
        square_nonneg(from_nat[Real](d))
        Real.0 <= from_nat[Real](d) * from_nat[Real](d)
        le_add_nonneg(Real.1, from_nat[Real](d) * from_nat[Real](d))
        Real.1 <= Real.1 + from_nat[Real](d) * from_nat[Real](d)
        add_lte_add(Real.1,
            from_nat[Real](a) * from_nat[Real](a) + Real.1, Real.1,
            Real.1 + from_nat[Real](d) * from_nat[Real](d))
        Real.1 + Real.1 <= (from_nat[Real](a) * from_nat[Real](a) + Real.1) +
            (Real.1 + from_nat[Real](d) * from_nat[Real](d))
        (from_nat[Real](a) * from_nat[Real](a) + Real.1) +
            (Real.1 + from_nat[Real](d) * from_nat[Real](d)) =
            from_nat[Real](a) * from_nat[Real](a) + Real.1 + Real.1 +
            from_nat[Real](d) * from_nat[Real](d)
        le_eq_rewrite(Real.1 + Real.1,
            (from_nat[Real](a) * from_nat[Real](a) + Real.1) +
            (Real.1 + from_nat[Real](d) * from_nat[Real](d)),
            from_nat[Real](a) * from_nat[Real](a) + Real.1 + Real.1 +
            from_nat[Real](d) * from_nat[Real](d))
        Real.1 + Real.1 <= from_nat[Real](a) * from_nat[Real](a) + Real.1 + Real.1 +
            from_nat[Real](d) * from_nat[Real](d)
        le_eq_rewrite(Real.1 + Real.1,
            from_nat[Real](a) * from_nat[Real](a) + Real.1 + Real.1 +
            from_nat[Real](d) * from_nat[Real](d),
            from_nat[Real](a) * from_nat[Real](a) +
            from_nat[Real](b) * from_nat[Real](b) +
            from_nat[Real](c) * from_nat[Real](c) +
            from_nat[Real](d) * from_nat[Real](d))
        Real.1 + Real.1 <= from_nat[Real](a) * from_nat[Real](a) +
            from_nat[Real](b) * from_nat[Real](b) +
            from_nat[Real](c) * from_nat[Real](c) +
            from_nat[Real](d) * from_nat[Real](d)
        real_add_pos_lt(Real.1, Real.1)
        Real.1 < Real.1 + Real.1
        lt_imp_lte(Real.1, Real.1 + Real.1)
        Real.1 <= Real.1 + Real.1
        lt_of_lt_of_lte(Real.1, Real.1 + Real.1,
            from_nat[Real](a) * from_nat[Real](a) +
            from_nat[Real](b) * from_nat[Real](b) +
            from_nat[Real](c) * from_nat[Real](c) +
            from_nat[Real](d) * from_nat[Real](d))
        Real.1 < from_nat[Real](a) * from_nat[Real](a) +
            from_nat[Real](b) * from_nat[Real](b) +
            from_nat[Real](c) * from_nat[Real](c) +
            from_nat[Real](d) * from_nat[Real](d)
        from_nat[Real](a) * from_nat[Real](a) +
            from_nat[Real](b) * from_nat[Real](b) +
            from_nat[Real](c) * from_nat[Real](c) +
            from_nat[Real](d) * from_nat[Real](d) > Real.1
    }
}

/// If the second and fourth of the four naturals are one and the sum of
/// their squares is one, then false.
theorem four_sq_two_ones_gt_one_24(a: Nat, b: Nat, c: Nat, d: Nat) {
    b = Nat.1 and d = Nat.1 implies
    from_nat[Real](a) * from_nat[Real](a) +
    from_nat[Real](b) * from_nat[Real](b) +
    from_nat[Real](c) * from_nat[Real](c) +
    from_nat[Real](d) * from_nat[Real](d) > Real.1
} by {
    if b = Nat.1 and d = Nat.1 {
        from_nat_one[Real]
        from_nat[Real](Nat.1) = Real.1
        from_nat[Real](b) = Real.1
        from_nat[Real](d) = Real.1
        from_nat[Real](b) * from_nat[Real](b) = Real.1
        from_nat[Real](d) * from_nat[Real](d) = Real.1
        from_nat[Real](a) * from_nat[Real](a) +
            from_nat[Real](b) * from_nat[Real](b) +
            from_nat[Real](c) * from_nat[Real](c) +
            from_nat[Real](d) * from_nat[Real](d) =
            from_nat[Real](a) * from_nat[Real](a) + Real.1 +
            from_nat[Real](c) * from_nat[Real](c) + Real.1
        square_nonneg(from_nat[Real](a))
        Real.0 <= from_nat[Real](a) * from_nat[Real](a)
        add_lte_add(Real.1, Real.1, Real.0, from_nat[Real](a) * from_nat[Real](a))
        Real.1 + Real.0 <= Real.1 + from_nat[Real](a) * from_nat[Real](a)
        Real.1 <= Real.1 + from_nat[Real](a) * from_nat[Real](a)
        Real.1 + from_nat[Real](a) * from_nat[Real](a) =
            from_nat[Real](a) * from_nat[Real](a) + Real.1
        le_eq_rewrite(Real.1, Real.1 + from_nat[Real](a) * from_nat[Real](a),
            from_nat[Real](a) * from_nat[Real](a) + Real.1)
        Real.1 <= from_nat[Real](a) * from_nat[Real](a) + Real.1
        square_nonneg(from_nat[Real](c))
        Real.0 <= from_nat[Real](c) * from_nat[Real](c)
        add_lte_add(Real.1, Real.1, Real.0, from_nat[Real](c) * from_nat[Real](c))
        Real.1 + Real.0 <= Real.1 + from_nat[Real](c) * from_nat[Real](c)
        Real.1 <= Real.1 + from_nat[Real](c) * from_nat[Real](c)
        Real.1 + from_nat[Real](c) * from_nat[Real](c) =
            from_nat[Real](c) * from_nat[Real](c) + Real.1
        le_eq_rewrite(Real.1, Real.1 + from_nat[Real](c) * from_nat[Real](c),
            from_nat[Real](c) * from_nat[Real](c) + Real.1)
        Real.1 <= from_nat[Real](c) * from_nat[Real](c) + Real.1
        add_lte_add(Real.1,
            from_nat[Real](a) * from_nat[Real](a) + Real.1, Real.1,
            from_nat[Real](c) * from_nat[Real](c) + Real.1)
        Real.1 + Real.1 <= (from_nat[Real](a) * from_nat[Real](a) + Real.1) +
            (from_nat[Real](c) * from_nat[Real](c) + Real.1)
        (from_nat[Real](a) * from_nat[Real](a) + Real.1) +
            (from_nat[Real](c) * from_nat[Real](c) + Real.1) =
            from_nat[Real](a) * from_nat[Real](a) + Real.1 +
            from_nat[Real](c) * from_nat[Real](c) + Real.1
        le_eq_rewrite(Real.1 + Real.1,
            (from_nat[Real](a) * from_nat[Real](a) + Real.1) +
            (from_nat[Real](c) * from_nat[Real](c) + Real.1),
            from_nat[Real](a) * from_nat[Real](a) + Real.1 +
            from_nat[Real](c) * from_nat[Real](c) + Real.1)
        Real.1 + Real.1 <= from_nat[Real](a) * from_nat[Real](a) + Real.1 +
            from_nat[Real](c) * from_nat[Real](c) + Real.1
        le_eq_rewrite(Real.1 + Real.1,
            from_nat[Real](a) * from_nat[Real](a) + Real.1 +
            from_nat[Real](c) * from_nat[Real](c) + Real.1,
            from_nat[Real](a) * from_nat[Real](a) +
            from_nat[Real](b) * from_nat[Real](b) +
            from_nat[Real](c) * from_nat[Real](c) +
            from_nat[Real](d) * from_nat[Real](d))
        Real.1 + Real.1 <= from_nat[Real](a) * from_nat[Real](a) +
            from_nat[Real](b) * from_nat[Real](b) +
            from_nat[Real](c) * from_nat[Real](c) +
            from_nat[Real](d) * from_nat[Real](d)
        real_add_pos_lt(Real.1, Real.1)
        Real.1 < Real.1 + Real.1
        lt_imp_lte(Real.1, Real.1 + Real.1)
        Real.1 <= Real.1 + Real.1
        lt_of_lt_of_lte(Real.1, Real.1 + Real.1,
            from_nat[Real](a) * from_nat[Real](a) +
            from_nat[Real](b) * from_nat[Real](b) +
            from_nat[Real](c) * from_nat[Real](c) +
            from_nat[Real](d) * from_nat[Real](d))
        Real.1 < from_nat[Real](a) * from_nat[Real](a) +
            from_nat[Real](b) * from_nat[Real](b) +
            from_nat[Real](c) * from_nat[Real](c) +
            from_nat[Real](d) * from_nat[Real](d)
        from_nat[Real](a) * from_nat[Real](a) +
            from_nat[Real](b) * from_nat[Real](b) +
            from_nat[Real](c) * from_nat[Real](c) +
            from_nat[Real](d) * from_nat[Real](d) > Real.1
    }
}

/// If the third and fourth of the four naturals are one and the sum of
/// their squares is one, then false.
theorem four_sq_two_ones_gt_one_34(a: Nat, b: Nat, c: Nat, d: Nat) {
    c = Nat.1 and d = Nat.1 implies
    from_nat[Real](a) * from_nat[Real](a) +
    from_nat[Real](b) * from_nat[Real](b) +
    from_nat[Real](c) * from_nat[Real](c) +
    from_nat[Real](d) * from_nat[Real](d) > Real.1
} by {
    if c = Nat.1 and d = Nat.1 {
        from_nat_one[Real]
        from_nat[Real](Nat.1) = Real.1
        from_nat[Real](c) = Real.1
        from_nat[Real](d) = Real.1
        from_nat[Real](c) * from_nat[Real](c) = Real.1
        from_nat[Real](d) * from_nat[Real](d) = Real.1
        from_nat[Real](a) * from_nat[Real](a) +
            from_nat[Real](b) * from_nat[Real](b) +
            from_nat[Real](c) * from_nat[Real](c) +
            from_nat[Real](d) * from_nat[Real](d) =
            from_nat[Real](a) * from_nat[Real](a) +
            from_nat[Real](b) * from_nat[Real](b) + Real.1 + Real.1
        two_sq_nonneg(from_nat[Real](a), from_nat[Real](b))
        from_nat[Real](a) * from_nat[Real](a) +
            from_nat[Real](b) * from_nat[Real](b) >= Real.0
        le_add_nonneg(from_nat[Real](a) * from_nat[Real](a) +
            from_nat[Real](b) * from_nat[Real](b), Real.1 + Real.1)
        from_nat[Real](a) * from_nat[Real](a) + from_nat[Real](b) * from_nat[Real](b) <= from_nat[Real](a) * from_nat[Real](a) + from_nat[Real](b) * from_nat[Real](b) + Real.1 + Real.1
        add_lte_add(Real.0,
            from_nat[Real](a) * from_nat[Real](a) +
            from_nat[Real](b) * from_nat[Real](b), Real.1, Real.1)
        Real.0 + Real.1 <= from_nat[Real](a) * from_nat[Real](a) +
            from_nat[Real](b) * from_nat[Real](b) + Real.1
        Real.1 <= from_nat[Real](a) * from_nat[Real](a) +
            from_nat[Real](b) * from_nat[Real](b) + Real.1
        add_lte_add(Real.1,
            from_nat[Real](a) * from_nat[Real](a) +
            from_nat[Real](b) * from_nat[Real](b) + Real.1, Real.1, Real.1)
        Real.1 + Real.1 <= from_nat[Real](a) * from_nat[Real](a) +
            from_nat[Real](b) * from_nat[Real](b) + Real.1 + Real.1
        le_eq_rewrite(Real.1 + Real.1,
            from_nat[Real](a) * from_nat[Real](a) +
            from_nat[Real](b) * from_nat[Real](b) + Real.1 + Real.1,
            from_nat[Real](a) * from_nat[Real](a) +
            from_nat[Real](b) * from_nat[Real](b) +
            from_nat[Real](c) * from_nat[Real](c) +
            from_nat[Real](d) * from_nat[Real](d))
        Real.1 + Real.1 <= from_nat[Real](a) * from_nat[Real](a) +
            from_nat[Real](b) * from_nat[Real](b) +
            from_nat[Real](c) * from_nat[Real](c) +
            from_nat[Real](d) * from_nat[Real](d)
        real_add_pos_lt(Real.1, Real.1)
        Real.1 < Real.1 + Real.1
        lt_imp_lte(Real.1, Real.1 + Real.1)
        Real.1 <= Real.1 + Real.1
        lt_of_lt_of_lte(Real.1, Real.1 + Real.1,
            from_nat[Real](a) * from_nat[Real](a) +
            from_nat[Real](b) * from_nat[Real](b) +
            from_nat[Real](c) * from_nat[Real](c) +
            from_nat[Real](d) * from_nat[Real](d))
        Real.1 < from_nat[Real](a) * from_nat[Real](a) +
            from_nat[Real](b) * from_nat[Real](b) +
            from_nat[Real](c) * from_nat[Real](c) +
            from_nat[Real](d) * from_nat[Real](d)
        from_nat[Real](a) * from_nat[Real](a) +
            from_nat[Real](b) * from_nat[Real](b) +
            from_nat[Real](c) * from_nat[Real](c) +
            from_nat[Real](d) * from_nat[Real](d) > Real.1
    }
}



/// A square of a real is at most the sum of four squares, with the square
/// in the second position.
theorem sq_le_sum4_b(a: Real, b: Real, c: Real, d: Real) {
    b * b <= a * a + b * b + c * c + d * d
} by {
    sq_le_sum4(b, a, c, d)
    b * b <= b * b + a * a + c * c + d * d
    b * b + a * a + c * c + d * d = a * a + b * b + c * c + d * d
    le_eq_rewrite(b * b, b * b + a * a + c * c + d * d, a * a + b * b + c * c + d * d)
    b * b <= a * a + b * b + c * c + d * d
}

/// A square of a real is at most the sum of four squares, with the square
/// in the third position.
theorem sq_le_sum4_c(a: Real, b: Real, c: Real, d: Real) {
    c * c <= a * a + b * b + c * c + d * d
} by {
    sq_le_sum4(c, a, b, d)
    c * c <= c * c + a * a + b * b + d * d
    c * c + a * a + b * b + d * d = a * a + b * b + c * c + d * d
    le_eq_rewrite(c * c, c * c + a * a + b * b + d * d, a * a + b * b + c * c + d * d)
    c * c <= a * a + b * b + c * c + d * d
}

/// A square of a real is at most the sum of four squares, with the square
/// in the fourth position.
theorem sq_le_sum4_d(a: Real, b: Real, c: Real, d: Real) {
    d * d <= a * a + b * b + c * c + d * d
} by {
    sq_le_sum4(d, a, b, c)
    d * d <= d * d + a * a + b * b + c * c
    d * d + a * a + b * b + c * c = a * a + d * d + b * b + c * c
    a * a + d * d + b * b + c * c = a * a + b * b + d * d + c * c
    d * d + c * c = c * c + d * d
    a * a + b * b + (d * d + c * c) = a * a + b * b + (c * c + d * d)
    a * a + b * b + d * d + c * c = a * a + b * b + (d * d + c * c)
    a * a + b * b + (c * c + d * d) = a * a + b * b + c * c + d * d
    a * a + b * b + d * d + c * c = a * a + b * b + c * c + d * d
    le_eq_rewrite(d * d, d * d + a * a + b * b + c * c, a * a + b * b + c * c + d * d)
    d * d <= a * a + b * b + c * c + d * d
}

/// If two of the four naturals are one, the sum of their squares is not one:
/// a = 1 and b = 1 are mutually exclusive under the unit-sum hypothesis.
theorem four_sq_not_both_12(a: Nat, b: Nat, c: Nat, d: Nat) {
    from_nat[Real](a) * from_nat[Real](a) +
    from_nat[Real](b) * from_nat[Real](b) +
    from_nat[Real](c) * from_nat[Real](c) +
    from_nat[Real](d) * from_nat[Real](d) = Real.1
    implies not (a = Nat.1 and b = Nat.1)
} by {
    if from_nat[Real](a) * from_nat[Real](a) +
        from_nat[Real](b) * from_nat[Real](b) +
        from_nat[Real](c) * from_nat[Real](c) +
        from_nat[Real](d) * from_nat[Real](d) = Real.1 {
        if a = Nat.1 and b = Nat.1 {
            four_sq_two_ones_gt_one(a, b, c, d)
            a = Nat.1 and b = Nat.1
            from_nat[Real](a) * from_nat[Real](a) +
            from_nat[Real](b) * from_nat[Real](b) +
            from_nat[Real](c) * from_nat[Real](c) +
            from_nat[Real](d) * from_nat[Real](d) > Real.1
            lt_of_lt_of_eq(Real.1,
                from_nat[Real](a) * from_nat[Real](a) +
                from_nat[Real](b) * from_nat[Real](b) +
                from_nat[Real](c) * from_nat[Real](c) +
                from_nat[Real](d) * from_nat[Real](d), Real.1)
            Real.1 < Real.1
            lt_not_ref(Real.1)
            false
        }
        not (a = Nat.1 and b = Nat.1)
    }
}

/// a = 1 and c = 1 are mutually exclusive under the unit-sum hypothesis.
theorem four_sq_not_both_13(a: Nat, b: Nat, c: Nat, d: Nat) {
    from_nat[Real](a) * from_nat[Real](a) +
    from_nat[Real](b) * from_nat[Real](b) +
    from_nat[Real](c) * from_nat[Real](c) +
    from_nat[Real](d) * from_nat[Real](d) = Real.1
    implies not (a = Nat.1 and c = Nat.1)
} by {
    if from_nat[Real](a) * from_nat[Real](a) +
        from_nat[Real](b) * from_nat[Real](b) +
        from_nat[Real](c) * from_nat[Real](c) +
        from_nat[Real](d) * from_nat[Real](d) = Real.1 {
        if a = Nat.1 and c = Nat.1 {
            four_sq_two_ones_gt_one_13(a, b, c, d)
            a = Nat.1 and c = Nat.1
            from_nat[Real](a) * from_nat[Real](a) +
            from_nat[Real](b) * from_nat[Real](b) +
            from_nat[Real](c) * from_nat[Real](c) +
            from_nat[Real](d) * from_nat[Real](d) > Real.1
            lt_of_lt_of_eq(Real.1,
                from_nat[Real](a) * from_nat[Real](a) +
                from_nat[Real](b) * from_nat[Real](b) +
                from_nat[Real](c) * from_nat[Real](c) +
                from_nat[Real](d) * from_nat[Real](d), Real.1)
            Real.1 < Real.1
            lt_not_ref(Real.1)
            false
        }
        not (a = Nat.1 and c = Nat.1)
    }
}

/// a = 1 and d = 1 are mutually exclusive under the unit-sum hypothesis.
theorem four_sq_not_both_14(a: Nat, b: Nat, c: Nat, d: Nat) {
    from_nat[Real](a) * from_nat[Real](a) +
    from_nat[Real](b) * from_nat[Real](b) +
    from_nat[Real](c) * from_nat[Real](c) +
    from_nat[Real](d) * from_nat[Real](d) = Real.1
    implies not (a = Nat.1 and d = Nat.1)
} by {
    if from_nat[Real](a) * from_nat[Real](a) +
        from_nat[Real](b) * from_nat[Real](b) +
        from_nat[Real](c) * from_nat[Real](c) +
        from_nat[Real](d) * from_nat[Real](d) = Real.1 {
        if a = Nat.1 and d = Nat.1 {
            four_sq_two_ones_gt_one_14(a, b, c, d)
            a = Nat.1 and d = Nat.1
            from_nat[Real](a) * from_nat[Real](a) +
            from_nat[Real](b) * from_nat[Real](b) +
            from_nat[Real](c) * from_nat[Real](c) +
            from_nat[Real](d) * from_nat[Real](d) > Real.1
            lt_of_lt_of_eq(Real.1,
                from_nat[Real](a) * from_nat[Real](a) +
                from_nat[Real](b) * from_nat[Real](b) +
                from_nat[Real](c) * from_nat[Real](c) +
                from_nat[Real](d) * from_nat[Real](d), Real.1)
            Real.1 < Real.1
            lt_not_ref(Real.1)
            false
        }
        not (a = Nat.1 and d = Nat.1)
    }
}

/// b = 1 and c = 1 are mutually exclusive under the unit-sum hypothesis.
theorem four_sq_not_both_23(a: Nat, b: Nat, c: Nat, d: Nat) {
    from_nat[Real](a) * from_nat[Real](a) +
    from_nat[Real](b) * from_nat[Real](b) +
    from_nat[Real](c) * from_nat[Real](c) +
    from_nat[Real](d) * from_nat[Real](d) = Real.1
    implies not (b = Nat.1 and c = Nat.1)
} by {
    if from_nat[Real](a) * from_nat[Real](a) +
        from_nat[Real](b) * from_nat[Real](b) +
        from_nat[Real](c) * from_nat[Real](c) +
        from_nat[Real](d) * from_nat[Real](d) = Real.1 {
        if b = Nat.1 and c = Nat.1 {
            four_sq_two_ones_gt_one_23(a, b, c, d)
            b = Nat.1 and c = Nat.1
            from_nat[Real](a) * from_nat[Real](a) +
            from_nat[Real](b) * from_nat[Real](b) +
            from_nat[Real](c) * from_nat[Real](c) +
            from_nat[Real](d) * from_nat[Real](d) > Real.1
            lt_of_lt_of_eq(Real.1,
                from_nat[Real](a) * from_nat[Real](a) +
                from_nat[Real](b) * from_nat[Real](b) +
                from_nat[Real](c) * from_nat[Real](c) +
                from_nat[Real](d) * from_nat[Real](d), Real.1)
            Real.1 < Real.1
            lt_not_ref(Real.1)
            false
        }
        not (b = Nat.1 and c = Nat.1)
    }
}

/// b = 1 and d = 1 are mutually exclusive under the unit-sum hypothesis.
theorem four_sq_not_both_24(a: Nat, b: Nat, c: Nat, d: Nat) {
    from_nat[Real](a) * from_nat[Real](a) +
    from_nat[Real](b) * from_nat[Real](b) +
    from_nat[Real](c) * from_nat[Real](c) +
    from_nat[Real](d) * from_nat[Real](d) = Real.1
    implies not (b = Nat.1 and d = Nat.1)
} by {
    if from_nat[Real](a) * from_nat[Real](a) +
        from_nat[Real](b) * from_nat[Real](b) +
        from_nat[Real](c) * from_nat[Real](c) +
        from_nat[Real](d) * from_nat[Real](d) = Real.1 {
        if b = Nat.1 and d = Nat.1 {
            four_sq_two_ones_gt_one_24(a, b, c, d)
            b = Nat.1 and d = Nat.1
            from_nat[Real](a) * from_nat[Real](a) +
            from_nat[Real](b) * from_nat[Real](b) +
            from_nat[Real](c) * from_nat[Real](c) +
            from_nat[Real](d) * from_nat[Real](d) > Real.1
            lt_of_lt_of_eq(Real.1,
                from_nat[Real](a) * from_nat[Real](a) +
                from_nat[Real](b) * from_nat[Real](b) +
                from_nat[Real](c) * from_nat[Real](c) +
                from_nat[Real](d) * from_nat[Real](d), Real.1)
            Real.1 < Real.1
            lt_not_ref(Real.1)
            false
        }
        not (b = Nat.1 and d = Nat.1)
    }
}

/// c = 1 and d = 1 are mutually exclusive under the unit-sum hypothesis.
theorem four_sq_not_both_34(a: Nat, b: Nat, c: Nat, d: Nat) {
    from_nat[Real](a) * from_nat[Real](a) +
    from_nat[Real](b) * from_nat[Real](b) +
    from_nat[Real](c) * from_nat[Real](c) +
    from_nat[Real](d) * from_nat[Real](d) = Real.1
    implies not (c = Nat.1 and d = Nat.1)
} by {
    if from_nat[Real](a) * from_nat[Real](a) +
        from_nat[Real](b) * from_nat[Real](b) +
        from_nat[Real](c) * from_nat[Real](c) +
        from_nat[Real](d) * from_nat[Real](d) = Real.1 {
        if c = Nat.1 and d = Nat.1 {
            four_sq_two_ones_gt_one_34(a, b, c, d)
            c = Nat.1 and d = Nat.1
            from_nat[Real](a) * from_nat[Real](a) +
            from_nat[Real](b) * from_nat[Real](b) +
            from_nat[Real](c) * from_nat[Real](c) +
            from_nat[Real](d) * from_nat[Real](d) > Real.1
            lt_of_lt_of_eq(Real.1,
                from_nat[Real](a) * from_nat[Real](a) +
                from_nat[Real](b) * from_nat[Real](b) +
                from_nat[Real](c) * from_nat[Real](c) +
                from_nat[Real](d) * from_nat[Real](d), Real.1)
            Real.1 < Real.1
            lt_not_ref(Real.1)
            false
        }
        not (c = Nat.1 and d = Nat.1)
    }
}
/// Each coordinate square is at most the total sum, hence at most one, so
/// each coordinate is zero or one; any two ones already exceed one, and all
/// zeroes fall short, so exactly one coordinate is one.
theorem four_sq_eq_one(a: Nat, b: Nat, c: Nat, d: Nat) {
    from_nat[Real](a) * from_nat[Real](a) +
    from_nat[Real](b) * from_nat[Real](b) +
    from_nat[Real](c) * from_nat[Real](c) +
    from_nat[Real](d) * from_nat[Real](d) = Real.1
    implies
    (a = Nat.1 and b = Nat.0 and c = Nat.0 and d = Nat.0) or
    (a = Nat.0 and b = Nat.1 and c = Nat.0 and d = Nat.0) or
    (a = Nat.0 and b = Nat.0 and c = Nat.1 and d = Nat.0) or
    (a = Nat.0 and b = Nat.0 and c = Nat.0 and d = Nat.1)
} by {
    if from_nat[Real](a) * from_nat[Real](a) +
        from_nat[Real](b) * from_nat[Real](b) +
        from_nat[Real](c) * from_nat[Real](c) +
        from_nat[Real](d) * from_nat[Real](d) = Real.1 {
        sq_le_sum4(from_nat[Real](a), from_nat[Real](b), from_nat[Real](c), from_nat[Real](d))
        from_nat[Real](a) * from_nat[Real](a) <= from_nat[Real](a) * from_nat[Real](a) +
        from_nat[Real](b) * from_nat[Real](b) +
        from_nat[Real](c) * from_nat[Real](c) +
        from_nat[Real](d) * from_nat[Real](d)
        le_eq_rewrite(from_nat[Real](a) * from_nat[Real](a),
            from_nat[Real](a) * from_nat[Real](a) +
        from_nat[Real](b) * from_nat[Real](b) +
        from_nat[Real](c) * from_nat[Real](c) +
        from_nat[Real](d) * from_nat[Real](d), Real.1)
        from_nat[Real](a) * from_nat[Real](a) <= Real.1
        from_nat_sq_le_one_imp_le_one(a)
        a <= Nat.1
        nat_le_one_zero_or_one(a)
        a = Nat.0 or a = Nat.1
        sq_le_sum4_b(from_nat[Real](a), from_nat[Real](b), from_nat[Real](c), from_nat[Real](d))
        from_nat[Real](b) * from_nat[Real](b) <= from_nat[Real](a) * from_nat[Real](a) +
        from_nat[Real](b) * from_nat[Real](b) +
        from_nat[Real](c) * from_nat[Real](c) +
        from_nat[Real](d) * from_nat[Real](d)
        le_eq_rewrite(from_nat[Real](b) * from_nat[Real](b),
            from_nat[Real](a) * from_nat[Real](a) +
        from_nat[Real](b) * from_nat[Real](b) +
        from_nat[Real](c) * from_nat[Real](c) +
        from_nat[Real](d) * from_nat[Real](d), Real.1)
        from_nat[Real](b) * from_nat[Real](b) <= Real.1
        from_nat_sq_le_one_imp_le_one(b)
        b <= Nat.1
        nat_le_one_zero_or_one(b)
        b = Nat.0 or b = Nat.1
        sq_le_sum4_c(from_nat[Real](a), from_nat[Real](b), from_nat[Real](c), from_nat[Real](d))
        from_nat[Real](c) * from_nat[Real](c) <= from_nat[Real](a) * from_nat[Real](a) +
        from_nat[Real](b) * from_nat[Real](b) +
        from_nat[Real](c) * from_nat[Real](c) +
        from_nat[Real](d) * from_nat[Real](d)
        le_eq_rewrite(from_nat[Real](c) * from_nat[Real](c),
            from_nat[Real](a) * from_nat[Real](a) +
        from_nat[Real](b) * from_nat[Real](b) +
        from_nat[Real](c) * from_nat[Real](c) +
        from_nat[Real](d) * from_nat[Real](d), Real.1)
        from_nat[Real](c) * from_nat[Real](c) <= Real.1
        from_nat_sq_le_one_imp_le_one(c)
        c <= Nat.1
        nat_le_one_zero_or_one(c)
        c = Nat.0 or c = Nat.1
        sq_le_sum4_d(from_nat[Real](a), from_nat[Real](b), from_nat[Real](c), from_nat[Real](d))
        from_nat[Real](d) * from_nat[Real](d) <= from_nat[Real](a) * from_nat[Real](a) +
        from_nat[Real](b) * from_nat[Real](b) +
        from_nat[Real](c) * from_nat[Real](c) +
        from_nat[Real](d) * from_nat[Real](d)
        le_eq_rewrite(from_nat[Real](d) * from_nat[Real](d),
            from_nat[Real](a) * from_nat[Real](a) +
        from_nat[Real](b) * from_nat[Real](b) +
        from_nat[Real](c) * from_nat[Real](c) +
        from_nat[Real](d) * from_nat[Real](d), Real.1)
        from_nat[Real](d) * from_nat[Real](d) <= Real.1
        from_nat_sq_le_one_imp_le_one(d)
        d <= Nat.1
        nat_le_one_zero_or_one(d)
        d = Nat.0 or d = Nat.1
        four_sq_not_both_12(a, b, c, d)
        from_nat[Real](a) * from_nat[Real](a) +
        from_nat[Real](b) * from_nat[Real](b) +
        from_nat[Real](c) * from_nat[Real](c) +
        from_nat[Real](d) * from_nat[Real](d) = Real.1
        not (a = Nat.1 and b = Nat.1)
        four_sq_not_both_13(a, b, c, d)
        from_nat[Real](a) * from_nat[Real](a) +
        from_nat[Real](b) * from_nat[Real](b) +
        from_nat[Real](c) * from_nat[Real](c) +
        from_nat[Real](d) * from_nat[Real](d) = Real.1
        not (a = Nat.1 and c = Nat.1)
        four_sq_not_both_14(a, b, c, d)
        from_nat[Real](a) * from_nat[Real](a) +
        from_nat[Real](b) * from_nat[Real](b) +
        from_nat[Real](c) * from_nat[Real](c) +
        from_nat[Real](d) * from_nat[Real](d) = Real.1
        not (a = Nat.1 and d = Nat.1)
        four_sq_not_both_23(a, b, c, d)
        from_nat[Real](a) * from_nat[Real](a) +
        from_nat[Real](b) * from_nat[Real](b) +
        from_nat[Real](c) * from_nat[Real](c) +
        from_nat[Real](d) * from_nat[Real](d) = Real.1
        not (b = Nat.1 and c = Nat.1)
        four_sq_not_both_24(a, b, c, d)
        from_nat[Real](a) * from_nat[Real](a) +
        from_nat[Real](b) * from_nat[Real](b) +
        from_nat[Real](c) * from_nat[Real](c) +
        from_nat[Real](d) * from_nat[Real](d) = Real.1
        not (b = Nat.1 and d = Nat.1)
        four_sq_not_both_34(a, b, c, d)
        from_nat[Real](a) * from_nat[Real](a) +
        from_nat[Real](b) * from_nat[Real](b) +
        from_nat[Real](c) * from_nat[Real](c) +
        from_nat[Real](d) * from_nat[Real](d) = Real.1
        not (c = Nat.1 and d = Nat.1)
        if a = Nat.0 {
            if b = Nat.0 {
                if c = Nat.0 {
                    if d = Nat.0 {
                        from_nat_zero[Real]
                        from_nat[Real](Nat.0) = Real.0
                        a = Nat.0
                        b = Nat.0
                        c = Nat.0
                        d = Nat.0
                        from_nat[Real](a) = Real.0
                        from_nat[Real](b) = Real.0
                        from_nat[Real](c) = Real.0
                        from_nat[Real](d) = Real.0
                        from_nat[Real](a) * from_nat[Real](a) +
        from_nat[Real](b) * from_nat[Real](b) +
        from_nat[Real](c) * from_nat[Real](c) +
        from_nat[Real](d) * from_nat[Real](d) =
                            Real.0 * Real.0 + Real.0 * Real.0 + Real.0 * Real.0 + Real.0 * Real.0
                        Real.0 * Real.0 + Real.0 * Real.0 + Real.0 * Real.0 + Real.0 * Real.0 = Real.0
                        from_nat[Real](a) * from_nat[Real](a) +
        from_nat[Real](b) * from_nat[Real](b) +
        from_nat[Real](c) * from_nat[Real](c) +
        from_nat[Real](d) * from_nat[Real](d) = Real.0
                        from_nat[Real](a) * from_nat[Real](a) +
        from_nat[Real](b) * from_nat[Real](b) +
        from_nat[Real](c) * from_nat[Real](c) +
        from_nat[Real](d) * from_nat[Real](d) = Real.1
                        Real.0 = Real.1
                        real_zero_lt_one
                        Real.0 < Real.1
                        Real.1 < Real.1
                        lt_not_ref(Real.1)
                        false
                    }
                    if d != Nat.0 {
                        Nat.0 < d
                        lt_imp_lte_suc(Nat.0, d)
                        Nat.1 <= d
                        lte_antisymm(d, Nat.1)
                        d = Nat.1
                        a = Nat.0 and b = Nat.0 and c = Nat.0 and d = Nat.1
                        (a = Nat.1 and b = Nat.0 and c = Nat.0 and d = Nat.0) or
                        (a = Nat.0 and b = Nat.1 and c = Nat.0 and d = Nat.0) or
                        (a = Nat.0 and b = Nat.0 and c = Nat.1 and d = Nat.0) or
                        (a = Nat.0 and b = Nat.0 and c = Nat.0 and d = Nat.1)
                    }
                    (a = Nat.1 and b = Nat.0 and c = Nat.0 and d = Nat.0) or
                    (a = Nat.0 and b = Nat.1 and c = Nat.0 and d = Nat.0) or
                    (a = Nat.0 and b = Nat.0 and c = Nat.1 and d = Nat.0) or
                    (a = Nat.0 and b = Nat.0 and c = Nat.0 and d = Nat.1)
                }
                if c != Nat.0 {
                    Nat.0 < c
                    lt_imp_lte_suc(Nat.0, c)
                    Nat.1 <= c
                    lte_antisymm(c, Nat.1)
                    c = Nat.1
                    if d = Nat.0 {
                        a = Nat.0 and b = Nat.0 and c = Nat.1 and d = Nat.0
                        (a = Nat.1 and b = Nat.0 and c = Nat.0 and d = Nat.0) or
                        (a = Nat.0 and b = Nat.1 and c = Nat.0 and d = Nat.0) or
                        (a = Nat.0 and b = Nat.0 and c = Nat.1 and d = Nat.0) or
                        (a = Nat.0 and b = Nat.0 and c = Nat.0 and d = Nat.1)
                    }
                    if d != Nat.0 {
                        Nat.0 < d
                        lt_imp_lte_suc(Nat.0, d)
                        Nat.1 <= d
                        lte_antisymm(d, Nat.1)
                        d = Nat.1
                        c = Nat.1 and d = Nat.1
                        not (c = Nat.1 and d = Nat.1)
                        false
                    }
                    (a = Nat.1 and b = Nat.0 and c = Nat.0 and d = Nat.0) or
                    (a = Nat.0 and b = Nat.1 and c = Nat.0 and d = Nat.0) or
                    (a = Nat.0 and b = Nat.0 and c = Nat.1 and d = Nat.0) or
                    (a = Nat.0 and b = Nat.0 and c = Nat.0 and d = Nat.1)
                }
                (a = Nat.1 and b = Nat.0 and c = Nat.0 and d = Nat.0) or
                (a = Nat.0 and b = Nat.1 and c = Nat.0 and d = Nat.0) or
                (a = Nat.0 and b = Nat.0 and c = Nat.1 and d = Nat.0) or
                (a = Nat.0 and b = Nat.0 and c = Nat.0 and d = Nat.1)
            }
            if b != Nat.0 {
                Nat.0 < b
                lt_imp_lte_suc(Nat.0, b)
                Nat.1 <= b
                lte_antisymm(b, Nat.1)
                b = Nat.1
                if c = Nat.0 {
                    if d = Nat.0 {
                        a = Nat.0 and b = Nat.1 and c = Nat.0 and d = Nat.0
                        (a = Nat.1 and b = Nat.0 and c = Nat.0 and d = Nat.0) or
                        (a = Nat.0 and b = Nat.1 and c = Nat.0 and d = Nat.0) or
                        (a = Nat.0 and b = Nat.0 and c = Nat.1 and d = Nat.0) or
                        (a = Nat.0 and b = Nat.0 and c = Nat.0 and d = Nat.1)
                    }
                    if d != Nat.0 {
                        Nat.0 < d
                        lt_imp_lte_suc(Nat.0, d)
                        Nat.1 <= d
                        lte_antisymm(d, Nat.1)
                        d = Nat.1
                        b = Nat.1 and d = Nat.1
                        not (b = Nat.1 and d = Nat.1)
                        false
                    }
                    (a = Nat.1 and b = Nat.0 and c = Nat.0 and d = Nat.0) or
                    (a = Nat.0 and b = Nat.1 and c = Nat.0 and d = Nat.0) or
                    (a = Nat.0 and b = Nat.0 and c = Nat.1 and d = Nat.0) or
                    (a = Nat.0 and b = Nat.0 and c = Nat.0 and d = Nat.1)
                }
                if c != Nat.0 {
                    Nat.0 < c
                    lt_imp_lte_suc(Nat.0, c)
                    Nat.1 <= c
                    lte_antisymm(c, Nat.1)
                    c = Nat.1
                    b = Nat.1 and c = Nat.1
                    not (b = Nat.1 and c = Nat.1)
                    false
                }
                (a = Nat.1 and b = Nat.0 and c = Nat.0 and d = Nat.0) or
                (a = Nat.0 and b = Nat.1 and c = Nat.0 and d = Nat.0) or
                (a = Nat.0 and b = Nat.0 and c = Nat.1 and d = Nat.0) or
                (a = Nat.0 and b = Nat.0 and c = Nat.0 and d = Nat.1)
            }
            (a = Nat.1 and b = Nat.0 and c = Nat.0 and d = Nat.0) or
            (a = Nat.0 and b = Nat.1 and c = Nat.0 and d = Nat.0) or
            (a = Nat.0 and b = Nat.0 and c = Nat.1 and d = Nat.0) or
            (a = Nat.0 and b = Nat.0 and c = Nat.0 and d = Nat.1)
        }
        if a != Nat.0 {
            Nat.0 < a
            lt_imp_lte_suc(Nat.0, a)
            Nat.1 <= a
            lte_antisymm(a, Nat.1)
            a = Nat.1
            if b = Nat.0 {
                if c = Nat.0 {
                    if d = Nat.0 {
                        a = Nat.1 and b = Nat.0 and c = Nat.0 and d = Nat.0
                        (a = Nat.1 and b = Nat.0 and c = Nat.0 and d = Nat.0) or
                        (a = Nat.0 and b = Nat.1 and c = Nat.0 and d = Nat.0) or
                        (a = Nat.0 and b = Nat.0 and c = Nat.1 and d = Nat.0) or
                        (a = Nat.0 and b = Nat.0 and c = Nat.0 and d = Nat.1)
                    }
                    if d != Nat.0 {
                        Nat.0 < d
                        lt_imp_lte_suc(Nat.0, d)
                        Nat.1 <= d
                        lte_antisymm(d, Nat.1)
                        d = Nat.1
                        a = Nat.1 and d = Nat.1
                        not (a = Nat.1 and d = Nat.1)
                        false
                    }
                    (a = Nat.1 and b = Nat.0 and c = Nat.0 and d = Nat.0) or
                    (a = Nat.0 and b = Nat.1 and c = Nat.0 and d = Nat.0) or
                    (a = Nat.0 and b = Nat.0 and c = Nat.1 and d = Nat.0) or
                    (a = Nat.0 and b = Nat.0 and c = Nat.0 and d = Nat.1)
                }
                if c != Nat.0 {
                    Nat.0 < c
                    lt_imp_lte_suc(Nat.0, c)
                    Nat.1 <= c
                    lte_antisymm(c, Nat.1)
                    c = Nat.1
                    a = Nat.1 and c = Nat.1
                    not (a = Nat.1 and c = Nat.1)
                    false
                }
                (a = Nat.1 and b = Nat.0 and c = Nat.0 and d = Nat.0) or
                (a = Nat.0 and b = Nat.1 and c = Nat.0 and d = Nat.0) or
                (a = Nat.0 and b = Nat.0 and c = Nat.1 and d = Nat.0) or
                (a = Nat.0 and b = Nat.0 and c = Nat.0 and d = Nat.1)
            }
            if b != Nat.0 {
                Nat.0 < b
                lt_imp_lte_suc(Nat.0, b)
                Nat.1 <= b
                lte_antisymm(b, Nat.1)
                b = Nat.1
                a = Nat.1 and b = Nat.1
                not (a = Nat.1 and b = Nat.1)
                false
            }
            (a = Nat.1 and b = Nat.0 and c = Nat.0 and d = Nat.0) or
            (a = Nat.0 and b = Nat.1 and c = Nat.0 and d = Nat.0) or
            (a = Nat.0 and b = Nat.0 and c = Nat.1 and d = Nat.0) or
            (a = Nat.0 and b = Nat.0 and c = Nat.0 and d = Nat.1)
        }
        (a = Nat.1 and b = Nat.0 and c = Nat.0 and d = Nat.0) or
        (a = Nat.0 and b = Nat.1 and c = Nat.0 and d = Nat.0) or
        (a = Nat.0 and b = Nat.0 and c = Nat.1 and d = Nat.0) or
        (a = Nat.0 and b = Nat.0 and c = Nat.0 and d = Nat.1)
    }
}
theorem from_nat_real_inj(m: Nat, n: Nat) {
    from_nat[Real](m) = from_nat[Real](n) implies m = n
} by {
    if from_nat[Real](m) = from_nat[Real](n) {
        if m != n {
            lt_or_lte(m, n)
            if m < n {
                from_nat_lt_strict(m, n)
                from_nat[Real](m) < from_nat[Real](n)
                false
            }
            if n < m {
                from_nat_lt_strict(n, m)
                from_nat[Real](n) < from_nat[Real](m)
                false
            }
            m = n
            false
        }
        m = n
    }
}

/// The complex number with two natural coordinates.
define complex_of_nat_pair(p: Pair[Nat, Nat]) -> Complex {
    Complex.new(from_nat[Real](p.first), from_nat[Real](p.second))
}

/// The quaternion with four natural coordinates, from two pairs of naturals.
define quat_of_nat_pair_pair(p: Pair[Pair[Nat, Nat], Pair[Nat, Nat]]) -> Quat {
    Quat.new(complex_of_nat_pair(p.first), complex_of_nat_pair(p.second))
}

/// The n by n by n by n grid {0, ..., n - 1}⁴ as quaternions.
define grid(n: Nat) -> FiniteSet[Quat] {
    fs_image(finite_set_product(finite_set_product(range_set(n), range_set(n)),
        finite_set_product(range_set(n), range_set(n))), quat_of_nat_pair_pair)
}

/// The four coordinate unit vectors of R⁴ as quaternions.
let grid_e1: Quat = Quat.new(Complex.new(Real.1, Real.0), Complex.new(Real.0, Real.0))
let grid_e2: Quat = Quat.new(Complex.new(Real.0, Real.1), Complex.new(Real.0, Real.0))
let grid_e3: Quat = Quat.new(Complex.new(Real.0, Real.0), Complex.new(Real.1, Real.0))
let grid_e4: Quat = Quat.new(Complex.new(Real.0, Real.0), Complex.new(Real.0, Real.1))

/// The squared absolute value of the complex number (1, 0) is one.
theorem complex_one_zero_abs_squared {
    Complex.new(Real.1, Real.0).abs_squared = Real.1
} by {
    abs_squared_eq(Complex.new(Real.1, Real.0))
    Complex.new(Real.1, Real.0).abs_squared =
        Complex.new(Real.1, Real.0).re * Complex.new(Real.1, Real.0).re +
        Complex.new(Real.1, Real.0).im * Complex.new(Real.1, Real.0).im
    Complex.new(Real.1, Real.0).re = Real.1
    Complex.new(Real.1, Real.0).im = Real.0
    Complex.new(Real.1, Real.0).abs_squared = Real.1 * Real.1 + Real.0 * Real.0
    Real.1 * Real.1 + Real.0 * Real.0 = Real.1
    Complex.new(Real.1, Real.0).abs_squared = Real.1
}

/// The squared absolute value of the complex number (0, 0) is zero.
theorem complex_zero_zero_abs_squared {
    Complex.new(Real.0, Real.0).abs_squared = Real.0
} by {
    abs_squared_eq(Complex.new(Real.0, Real.0))
    Complex.new(Real.0, Real.0).abs_squared =
        Complex.new(Real.0, Real.0).re * Complex.new(Real.0, Real.0).re +
        Complex.new(Real.0, Real.0).im * Complex.new(Real.0, Real.0).im
    Complex.new(Real.0, Real.0).re = Real.0
    Complex.new(Real.0, Real.0).im = Real.0
    Complex.new(Real.0, Real.0).abs_squared = Real.0 * Real.0 + Real.0 * Real.0
    Real.0 * Real.0 + Real.0 * Real.0 = Real.0
    Complex.new(Real.0, Real.0).abs_squared = Real.0
}

/// The first coordinate unit vector has norm one.
theorem grid_e1_norm_one {
    quat_norm(grid_e1) = Real.1
} by {
    quat_norm(grid_e1) = norm2(grid_e1.first, grid_e1.second)
    norm2(grid_e1.first, grid_e1.second) =
        grid_e1.first.abs_squared + grid_e1.second.abs_squared
    grid_e1 = Quat.new(Complex.new(Real.1, Real.0), Complex.new(Real.0, Real.0))
    grid_e1.first = Complex.new(Real.1, Real.0)
    grid_e1.second = Complex.new(Real.0, Real.0)
    complex_one_zero_abs_squared
    Complex.new(Real.1, Real.0).abs_squared = Real.1
    complex_zero_zero_abs_squared
    Complex.new(Real.0, Real.0).abs_squared = Real.0
    grid_e1.first.abs_squared = Real.1
    grid_e1.second.abs_squared = Real.0
    grid_e1.first.abs_squared + grid_e1.second.abs_squared = Real.1 + Real.0
    Real.1 + Real.0 = Real.1
    quat_norm(grid_e1) = Real.1
}

/// The second coordinate unit vector has norm one.
theorem grid_e2_norm_one {
    quat_norm(grid_e2) = Real.1
} by {
    quat_norm(grid_e2) = norm2(grid_e2.first, grid_e2.second)
    norm2(grid_e2.first, grid_e2.second) =
        grid_e2.first.abs_squared + grid_e2.second.abs_squared
    grid_e2 = Quat.new(Complex.new(Real.0, Real.1), Complex.new(Real.0, Real.0))
    grid_e2.first = Complex.new(Real.0, Real.1)
    grid_e2.second = Complex.new(Real.0, Real.0)
    abs_squared_eq(grid_e2.first)
    grid_e2.first.abs_squared =
        grid_e2.first.re * grid_e2.first.re + grid_e2.first.im * grid_e2.first.im
    grid_e2.first.re = Real.0
    grid_e2.first.im = Real.1
    grid_e2.first.abs_squared = Real.0 * Real.0 + Real.1 * Real.1
    Real.0 * Real.0 + Real.1 * Real.1 = Real.1
    grid_e2.first.abs_squared = Real.1
    complex_zero_zero_abs_squared
    Complex.new(Real.0, Real.0).abs_squared = Real.0
    grid_e2.second.abs_squared = Real.0
    grid_e2.first.abs_squared + grid_e2.second.abs_squared = Real.1 + Real.0
    Real.1 + Real.0 = Real.1
    quat_norm(grid_e2) = Real.1
}

/// The third coordinate unit vector has norm one.
theorem grid_e3_norm_one {
    quat_norm(grid_e3) = Real.1
} by {
    quat_norm(grid_e3) = norm2(grid_e3.first, grid_e3.second)
    norm2(grid_e3.first, grid_e3.second) =
        grid_e3.first.abs_squared + grid_e3.second.abs_squared
    grid_e3 = Quat.new(Complex.new(Real.0, Real.0), Complex.new(Real.1, Real.0))
    grid_e3.first = Complex.new(Real.0, Real.0)
    grid_e3.second = Complex.new(Real.1, Real.0)
    complex_zero_zero_abs_squared
    Complex.new(Real.0, Real.0).abs_squared = Real.0
    grid_e3.first.abs_squared = Real.0
    abs_squared_eq(grid_e3.second)
    grid_e3.second.abs_squared =
        grid_e3.second.re * grid_e3.second.re + grid_e3.second.im * grid_e3.second.im
    grid_e3.second.re = Real.1
    grid_e3.second.im = Real.0
    grid_e3.second.abs_squared = Real.1 * Real.1 + Real.0 * Real.0
    Real.1 * Real.1 + Real.0 * Real.0 = Real.1
    grid_e3.second.abs_squared = Real.1
    grid_e3.first.abs_squared + grid_e3.second.abs_squared = Real.0 + Real.1
    Real.0 + Real.1 = Real.1
    quat_norm(grid_e3) = Real.1
}

/// The fourth coordinate unit vector has norm one.
theorem grid_e4_norm_one {
    quat_norm(grid_e4) = Real.1
} by {
    quat_norm(grid_e4) = norm2(grid_e4.first, grid_e4.second)
    norm2(grid_e4.first, grid_e4.second) =
        grid_e4.first.abs_squared + grid_e4.second.abs_squared
    grid_e4 = Quat.new(Complex.new(Real.0, Real.0), Complex.new(Real.0, Real.1))
    grid_e4.first = Complex.new(Real.0, Real.0)
    grid_e4.second = Complex.new(Real.0, Real.1)
    complex_zero_zero_abs_squared
    Complex.new(Real.0, Real.0).abs_squared = Real.0
    grid_e4.first.abs_squared = Real.0
    abs_squared_eq(grid_e4.second)
    grid_e4.second.abs_squared =
        grid_e4.second.re * grid_e4.second.re + grid_e4.second.im * grid_e4.second.im
    grid_e4.second.re = Real.0
    grid_e4.second.im = Real.1
    grid_e4.second.abs_squared = Real.0 * Real.0 + Real.1 * Real.1
    Real.0 * Real.0 + Real.1 * Real.1 = Real.1
    grid_e4.second.abs_squared = Real.1
    grid_e4.first.abs_squared + grid_e4.second.abs_squared = Real.0 + Real.1
    Real.0 + Real.1 = Real.1
    quat_norm(grid_e4) = Real.1
}

/// Translating by a unit vector produces a point at unit distance.
theorem grid_step_unit_dist(a: Quat, w: Quat) {
    quat_norm(w) = Real.1 implies quaternion_unit_distance(a, quat_add(a, w))
} by {
    if quat_norm(w) = Real.1 {
        quaternion_unit_distance(a, quat_add(a, w)) =
            (quat_dist_sq(a, quat_add(a, w)) = Real.1)
        quat_dist_sq_add_right(a, w)
        quat_dist_sq(a, quat_add(a, w)) = quat_norm(w)
        quat_dist_sq(a, quat_add(a, w)) = Real.1
        quaternion_unit_distance(a, quat_add(a, w))
    }
}

/// The diagonal e₁ + e₂ has squared norm two.
theorem grid_diag_norm_two {
    quat_norm(quat_add(grid_e1, grid_e2)) = Real.1 + Real.1
} by {
    quat_norm(quat_add(grid_e1, grid_e2)) =
        norm2(quat_add(grid_e1, grid_e2).first, quat_add(grid_e1, grid_e2).second)
    norm2(quat_add(grid_e1, grid_e2).first, quat_add(grid_e1, grid_e2).second) =
        quat_add(grid_e1, grid_e2).first.abs_squared +
        quat_add(grid_e1, grid_e2).second.abs_squared
    quat_add(grid_e1, grid_e2) =
        Quat.new(grid_e1.first + grid_e2.first, grid_e1.second + grid_e2.second)
    grid_e1 = Quat.new(Complex.new(Real.1, Real.0), Complex.new(Real.0, Real.0))
    grid_e2 = Quat.new(Complex.new(Real.0, Real.1), Complex.new(Real.0, Real.0))
    grid_e1.first = Complex.new(Real.1, Real.0)
    grid_e2.first = Complex.new(Real.0, Real.1)
    grid_e1.second = Complex.new(Real.0, Real.0)
    grid_e2.second = Complex.new(Real.0, Real.0)
    Complex.new(Real.1, Real.0) + Complex.new(Real.0, Real.1) =
        Complex.new(Real.1, Real.1)
    Complex.new(Real.0, Real.0) + Complex.new(Real.0, Real.0) =
        Complex.new(Real.0, Real.0)
    quat_add(grid_e1, grid_e2) =
        Quat.new(Complex.new(Real.1, Real.1), Complex.new(Real.0, Real.0))
    quat_add(grid_e1, grid_e2).first = Complex.new(Real.1, Real.1)
    quat_add(grid_e1, grid_e2).second = Complex.new(Real.0, Real.0)
    abs_squared_eq(Complex.new(Real.1, Real.1))
    Complex.new(Real.1, Real.1).abs_squared =
        Complex.new(Real.1, Real.1).re * Complex.new(Real.1, Real.1).re +
        Complex.new(Real.1, Real.1).im * Complex.new(Real.1, Real.1).im
    Complex.new(Real.1, Real.1).re = Real.1
    Complex.new(Real.1, Real.1).im = Real.1
    Complex.new(Real.1, Real.1).abs_squared = Real.1 * Real.1 + Real.1 * Real.1
    Real.1 * Real.1 + Real.1 * Real.1 = Real.1 + Real.1
    Complex.new(Real.1, Real.1).abs_squared = Real.1 + Real.1
    complex_zero_zero_abs_squared
    Complex.new(Real.0, Real.0).abs_squared = Real.0
    quat_add(grid_e1, grid_e2).first.abs_squared = Real.1 + Real.1
    quat_add(grid_e1, grid_e2).second.abs_squared = Real.0
    quat_add(grid_e1, grid_e2).first.abs_squared +
        quat_add(grid_e1, grid_e2).second.abs_squared = Real.1 + Real.1 + Real.0
    Real.1 + Real.1 + Real.0 = Real.1 + Real.1
    quat_norm(quat_add(grid_e1, grid_e2)) = Real.1 + Real.1
}

/// A diagonal ±eᵢ ± eⱼ is not a unit step: its squared length is two.
theorem grid_diag_not_unit {
    quat_norm(quat_add(grid_e1, grid_e2)) != Real.1
} by {
    grid_diag_norm_two
    quat_norm(quat_add(grid_e1, grid_e2)) = Real.1 + Real.1
    if quat_norm(quat_add(grid_e1, grid_e2)) = Real.1 {
        Real.1 + Real.1 = Real.1
        real_zero_lt_one
        Real.0 < Real.1
        real_add_pos_lt(Real.1, Real.1)
        Real.1 < Real.1 + Real.1
        Real.1 < Real.1
        lt_not_ref(Real.1)
        false
    }
    quat_norm(quat_add(grid_e1, grid_e2)) != Real.1
}

// The exact grid count that the lemmas above feed.  A unit pair of grid
// points differs by ±1 in exactly one coordinate (the four-squares
// characterization `four_sq_eq_one`), so the ordered unit pairs are exactly
// the pairs (x, x ± eᵢ) with both endpoints in the grid: 4 coordinates ×
// 2 signs × n³(n − 1) choices of the base point, where the coordinate in the
// direction of the step ranges over {0, ..., n − 2} (n − 1 values) and the
// other three coordinates are free (n³ values).  The count is then a direct
// enumeration of that product; it is recorded here as a statement:
//
//   theorem grid_ordered_unit_pairs_card(n: Nat) {
//       n != Nat.0 implies
//       fs_card(ordered_unit_pairs(grid(n))) = Nat.8 * n * n * n * (n - Nat.1)
//   }
//
//   theorem grid_nu(n: Nat) {
//       n != Nat.0 implies nu(grid(n)) = Nat.4 * n * n * n * (n - Nat.1)
//   }
//
