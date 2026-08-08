from nat import Nat
from list import List, map
from list import sum
from nat import divides_self, lt_not_ref, mul_to_zero, mul_cancel_left,
    gcd_divides_left, gcd_divides_right, cofactor, divides_symm, distrib_right
from pair import Pair, pair_eta
from data.list.list_pair_product import list_pair_product, list_pair_with_left, pair_with_left_fn,
    list_pair_product_contains_of_contains,
    list_pair_product_contains_imp_left, list_pair_product_contains_imp_right,
    list_pair_product_unique, list_pair_product_length,
    list_pair_product_map_mul_sum, pair_product_map_value
from number_theory.arithmetic_functions import nat_zero_arithmetic_fn, nat_dirichlet_unit_fn,
    nat_dirichlet_unit_fn_at_one, nat_dirichlet_unit_fn_off_one,
    arithmetic_fn_add, arithmetic_fn_add_apply, is_multiplicative_nat_fn,
    is_multiplicative_nat_fn_at_one, multiplicative_nat_fn_apply,
    nat_identity_arithmetic_fn, nat_mul_swap_middle
from algebra.add_semigroup import add_fn
from list import map_sum_add
from number_theory.divisor_sum import divisor_list, divisor_list_zero, divisor_list_one,
    divisors_up_to, divisors_up_to_zero, divisors_up_to_one,
    divisors_up_to_suc_yes, divisors_up_to_suc_no,
    divisor_list_contains_implies, divisor_list_contains_of, divisor_list_is_unique,
    nat_tau, nat_tau_zero, nat_tau_one, nat_sigma, nat_sigma_zero, nat_sigma_one,
    sum_map_nat_identity_arithmetic_fn_eq_sum
from number_theory.coprime import coprime_divides_of_divides_mul, coprime_of_divisors,
    coprime_zero_left_imp_one, coprime_zero_right_imp_one
from list import map_contains, map_contains_of_contains, map_length, map_map,
    map_add, sum_add, sum_scalar_mul, scalar_mul
from list import sum_map_of_pointwise, unique_same_contains_map_sum_eq
from list import is_permutation, permutation_preserves_length,
    unique_same_contains_imp_permutation
from data.basic.functions import compose
from data.finite.finite_fiber_partition import locally_injective_map_is_unique
numerals Nat

/// The unique cofactor `q` with `d * q = n` when `d` divides `n`, and zero
/// otherwise. Lets us write `n / d` as `divisor_quotient(n, d)` in arithmetic
/// arguments where `d` ranges over the divisors of `n`.
let divisor_quotient(n: Nat, d: Nat) -> q: Nat satisfy {
    if d.divides(n) {
        d * q = n
    } else {
        q = Nat.0
    }
} by {
    if d.divides(n) {
        let x: Nat satisfy { d * x = n }
        d * x = n
    } else {
        let x: Nat satisfy { x = Nat.0 }
    }
}

/// When `d` divides `n`, the divisor quotient produces the cofactor.
theorem divisor_quotient_cofactor(n: Nat, d: Nat) {
    d.divides(n) implies d * divisor_quotient(n, d) = n
}

/// One divides every natural number, so the divisor quotient at `d = 1` is `n`.
theorem divisor_quotient_one(n: Nat) {
    divisor_quotient(n, Nat.1) = n
} by {
    Nat.1 * n = n
    Nat.1.divides(n) = exists(c: Nat) { Nat.1 * c = n }
    Nat.1.divides(n)
    divisor_quotient_cofactor(n, Nat.1)
    Nat.1 * divisor_quotient(n, Nat.1) = n
    Nat.1 * divisor_quotient(n, Nat.1) = divisor_quotient(n, Nat.1)
}

/// The cofactor when dividing `n` by itself is `1`, for positive `n`.
theorem divisor_quotient_self(n: Nat) {
    Nat.0 < n implies divisor_quotient(n, n) = Nat.1
} by {
    if Nat.0 < n {
        divides_self(n)
        n.divides(n)
        divisor_quotient_cofactor(n, n)
        n * divisor_quotient(n, n) = n
        n * Nat.1 = n
        if divisor_quotient(n, n) != Nat.1 {
            n * divisor_quotient(n, n) != n * Nat.1
            false
        }
    }
}

/// The single-argument fiber of the Dirichlet convolution: at fixed `n`, this
/// is the function `d -> f(d) * g(n / d)`.
define dirichlet_term(f: Nat -> Nat, g: Nat -> Nat, n: Nat) -> (Nat -> Nat) {
    function(d: Nat) { f(d) * g(divisor_quotient(n, d)) }
}

/// Application of the Dirichlet term function.
theorem dirichlet_term_apply(f: Nat -> Nat, g: Nat -> Nat, n: Nat, d: Nat) {
    dirichlet_term(f, g, n)(d) = f(d) * g(divisor_quotient(n, d))
}

/// The Dirichlet convolution `dirichlet_convolve(f, g)(n) = sum_{d | n} f(d) * g(n / d)`.
define dirichlet_convolve(f: Nat -> Nat, g: Nat -> Nat) -> (Nat -> Nat) {
    function(n: Nat) { sum(map(divisor_list(n), dirichlet_term(f, g, n))) }
}

/// Application of the Dirichlet convolution unfolds to the divisor-list sum.
theorem dirichlet_convolve_apply(f: Nat -> Nat, g: Nat -> Nat, n: Nat) {
    dirichlet_convolve(f, g)(n) = sum(map(divisor_list(n), dirichlet_term(f, g, n)))
}

/// `dirichlet_convolve(f, g)(0) = 0`, since zero has no positive divisors.
theorem dirichlet_convolve_at_zero(f: Nat -> Nat, g: Nat -> Nat) {
    dirichlet_convolve(f, g)(Nat.0) = Nat.0
} by {
    divisor_list_zero
    divisor_list(Nat.0) = List.nil[Nat]
    map(List.nil[Nat], dirichlet_term(f, g, Nat.0)) = List.nil[Nat]
    sum(List.nil[Nat]) = Nat.0
    dirichlet_convolve(f, g)(Nat.0) = sum(map(divisor_list(Nat.0), dirichlet_term(f, g, Nat.0)))
    dirichlet_convolve(f, g)(Nat.0) = sum(List.nil[Nat])
}

/// `dirichlet_convolve(f, g)(1) = f(1) * g(1)`, since the only positive divisor
/// of one is one.
theorem dirichlet_convolve_at_one(f: Nat -> Nat, g: Nat -> Nat) {
    dirichlet_convolve(f, g)(Nat.1) = f(Nat.1) * g(Nat.1)
} by {
    divisor_list_one
    divisor_list(Nat.1) = List.cons(Nat.1, List.nil[Nat])
    let h: Nat -> Nat = dirichlet_term(f, g, Nat.1)
    map(List.cons(Nat.1, List.nil[Nat]), h) = List.cons(h(Nat.1), map(List.nil[Nat], h))
    map(List.nil[Nat], h) = List.nil[Nat]
    map(List.cons(Nat.1, List.nil[Nat]), h) = List.cons(h(Nat.1), List.nil[Nat])
    h(Nat.1) = f(Nat.1) * g(divisor_quotient(Nat.1, Nat.1))
    divisor_quotient_one(Nat.1)
    divisor_quotient(Nat.1, Nat.1) = Nat.1
    h(Nat.1) = f(Nat.1) * g(Nat.1)
    sum(List.cons(h(Nat.1), List.nil[Nat])) = h(Nat.1) + sum(List.nil[Nat])
    sum(List.nil[Nat]) = Nat.0
    h(Nat.1) + Nat.0 = h(Nat.1)
    sum(List.cons(h(Nat.1), List.nil[Nat])) = h(Nat.1)
    dirichlet_convolve(f, g)(Nat.1) = sum(map(divisor_list(Nat.1), dirichlet_term(f, g, Nat.1)))
    dirichlet_convolve(f, g)(Nat.1) = sum(map(List.cons(Nat.1, List.nil[Nat]), h))
    dirichlet_convolve(f, g)(Nat.1) = h(Nat.1)
}

/// Predicate for the zero-fn induction: `sum(map(xs, h)) = 0` whenever `h` is
/// identically zero.
define sum_map_zero_pred(h: Nat -> Nat) -> (List[Nat] -> Bool) {
    function(xs: List[Nat]) {
        forall(d: Nat) { h(d) = Nat.0 } implies sum(map(xs, h)) = Nat.0
    }
}

/// Base of the zero-fn induction.
theorem sum_map_zero_pred_nil(h: Nat -> Nat) {
    sum_map_zero_pred(h)(List.nil[Nat])
} by {
    map(List.nil[Nat], h) = List.nil[Nat]
    sum(List.nil[Nat]) = Nat.0
}

/// Step of the zero-fn induction.
theorem sum_map_zero_pred_step(h: Nat -> Nat, head: Nat, tail: List[Nat]) {
    sum_map_zero_pred(h)(tail) implies sum_map_zero_pred(h)(List.cons(head, tail))
} by {
    if sum_map_zero_pred(h)(tail) {
        if forall(d: Nat) { h(d) = Nat.0 } {
            map(List.cons(head, tail), h) = List.cons(h(head), map(tail, h))
            h(head) = Nat.0
            map(List.cons(head, tail), h) = List.cons(Nat.0, map(tail, h))
            sum(List.cons(Nat.0, map(tail, h))) = Nat.0 + sum(map(tail, h))
            sum(map(tail, h)) = Nat.0
            Nat.0 + Nat.0 = Nat.0
            sum(map(List.cons(head, tail), h)) = Nat.0
        }
    }
}

/// If a function `h` is identically zero, the sum of its image on any list is
/// zero. A general helper used to prove the zero annihilators.
theorem sum_map_zero_fn(l: List[Nat], h: Nat -> Nat) {
    forall(d: Nat) { h(d) = Nat.0 } implies sum(map(l, h)) = Nat.0
} by {
    sum_map_zero_pred_nil(h)
    forall(head: Nat, tail: List[Nat]) {
        if sum_map_zero_pred(h)(tail) {
            sum_map_zero_pred_step(h, head, tail)
        }
    }
    sum_map_zero_pred(h)(l)
}

/// The Dirichlet convolution annihilates on the left by the constant-zero
/// arithmetic function.
theorem dirichlet_convolve_zero_left(g: Nat -> Nat) {
    dirichlet_convolve(nat_zero_arithmetic_fn, g) = nat_zero_arithmetic_fn
} by {
    forall(n: Nat) {
        let h: Nat -> Nat = dirichlet_term(nat_zero_arithmetic_fn, g, n)
        forall(d: Nat) {
            h(d) = nat_zero_arithmetic_fn(d) * g(divisor_quotient(n, d))
            nat_zero_arithmetic_fn(d) = Nat.0
            h(d) = Nat.0 * g(divisor_quotient(n, d))
            Nat.0 * g(divisor_quotient(n, d)) = Nat.0
            h(d) = Nat.0
        }
        sum_map_zero_fn(divisor_list(n), h)
        sum(map(divisor_list(n), h)) = Nat.0
        dirichlet_convolve(nat_zero_arithmetic_fn, g)(n) =
            sum(map(divisor_list(n), dirichlet_term(nat_zero_arithmetic_fn, g, n)))
        dirichlet_convolve(nat_zero_arithmetic_fn, g)(n) = Nat.0
        nat_zero_arithmetic_fn(n) = Nat.0
        dirichlet_convolve(nat_zero_arithmetic_fn, g)(n) = nat_zero_arithmetic_fn(n)
    }
}

/// If `d` divides `n` and `d < n`, the cofactor `n / d` is not one.
theorem divisor_quotient_proper_not_one(n: Nat, d: Nat) {
    d.divides(n) and d < n implies divisor_quotient(n, d) != Nat.1
} by {
    if d.divides(n) and d < n {
        divisor_quotient_cofactor(n, d)
        d * divisor_quotient(n, d) = n
        d != n
        if divisor_quotient(n, d) = Nat.1 {
            d * Nat.1 = d
            d * divisor_quotient(n, d) = d * Nat.1
            d * divisor_quotient(n, d) = d
            d = n
            false
        }
    }
}

/// A proper divisor `d < n` of `n` has non-one cofactor, so its Dirichlet-unit
/// term vanishes.
theorem dirichlet_term_unit_proper_zero(f: Nat -> Nat, n: Nat, d: Nat) {
    d.divides(n) and d < n
        implies dirichlet_term(f, nat_dirichlet_unit_fn, n)(d) = Nat.0
} by {
    if d.divides(n) and d < n {
        divisor_quotient_proper_not_one(n, d)
        divisor_quotient(n, d) != Nat.1
        nat_dirichlet_unit_fn_off_one(divisor_quotient(n, d))
        nat_dirichlet_unit_fn(divisor_quotient(n, d)) = Nat.0
        dirichlet_term_apply(f, nat_dirichlet_unit_fn, n, d)
        dirichlet_term(f, nat_dirichlet_unit_fn, n)(d) =
            f(d) * nat_dirichlet_unit_fn(divisor_quotient(n, d))
        f(d) * nat_dirichlet_unit_fn(divisor_quotient(n, d)) = f(d) * Nat.0
        f(d) * Nat.0 = Nat.0
        dirichlet_term(f, nat_dirichlet_unit_fn, n)(d) = Nat.0
    }
}

/// Inductive predicate for the right-identity helper: at bound `k < n`, the
/// dirichlet-term sum over `divisors_up_to(n, k)` vanishes because every such
/// divisor produces a non-one cofactor for the Dirichlet unit.
define dirichlet_unit_right_below_pred(f: Nat -> Nat, n: Nat, k: Nat) -> Bool {
    k < n implies
        sum(map(divisors_up_to(n, k),
            dirichlet_term(f, nat_dirichlet_unit_fn, n))) = Nat.0
}

/// Base case for the right-identity helper.
theorem dirichlet_unit_right_below_base(f: Nat -> Nat, n: Nat) {
    dirichlet_unit_right_below_pred(f, n, Nat.0)
} by {
    divisors_up_to_zero(n)
    divisors_up_to(n, Nat.0) = List.nil[Nat]
    map(List.nil[Nat], dirichlet_term(f, nat_dirichlet_unit_fn, n)) = List.nil[Nat]
    sum(List.nil[Nat]) = Nat.0
}

/// Cons-step lemma: if a head value is zero and the tail sum is zero, the
/// cons-sum is zero.
theorem sum_cons_zero_head(h: Nat -> Nat, head: Nat, tail: List[Nat]) {
    h(head) = Nat.0 and sum(map(tail, h)) = Nat.0
        implies sum(map(List.cons(head, tail), h)) = Nat.0
} by {
    if h(head) = Nat.0 and sum(map(tail, h)) = Nat.0 {
        map(List.cons(head, tail), h) = List.cons(h(head), map(tail, h))
        h(head) = Nat.0
        map(List.cons(head, tail), h) = List.cons(Nat.0, map(tail, h))
        sum(List.cons(Nat.0, map(tail, h))) = Nat.0 + sum(map(tail, h))
        sum(map(tail, h)) = Nat.0
        Nat.0 + Nat.0 = Nat.0
        sum(map(List.cons(head, tail), h)) = Nat.0
    }
}

/// Step case with `k.suc` a divisor of `n`: the head term vanishes by
/// `dirichlet_term_unit_proper_zero`, so the sum is unchanged.
theorem dirichlet_unit_right_below_step_yes(f: Nat -> Nat, n: Nat, k: Nat) {
    k.suc < n and k.suc.divides(n) and
        sum(map(divisors_up_to(n, k),
            dirichlet_term(f, nat_dirichlet_unit_fn, n))) = Nat.0
        implies sum(map(divisors_up_to(n, k.suc),
            dirichlet_term(f, nat_dirichlet_unit_fn, n))) = Nat.0
} by {
    if k.suc < n and k.suc.divides(n) and
            sum(map(divisors_up_to(n, k),
                dirichlet_term(f, nat_dirichlet_unit_fn, n))) = Nat.0 {
        divisors_up_to_suc_yes(n, k)
        divisors_up_to(n, k.suc) = List.cons(k.suc, divisors_up_to(n, k))
        dirichlet_term_unit_proper_zero(f, n, k.suc)
        dirichlet_term(f, nat_dirichlet_unit_fn, n)(k.suc) = Nat.0
        sum_cons_zero_head(dirichlet_term(f, nat_dirichlet_unit_fn, n),
            k.suc, divisors_up_to(n, k))
        sum(map(List.cons(k.suc, divisors_up_to(n, k)),
            dirichlet_term(f, nat_dirichlet_unit_fn, n))) = Nat.0
        map(divisors_up_to(n, k.suc), dirichlet_term(f, nat_dirichlet_unit_fn, n)) =
            map(List.cons(k.suc, divisors_up_to(n, k)),
                dirichlet_term(f, nat_dirichlet_unit_fn, n))
        sum(map(divisors_up_to(n, k.suc),
            dirichlet_term(f, nat_dirichlet_unit_fn, n))) = Nat.0
    }
}

/// Step case with `k.suc` a non-divisor: the bounded list is unchanged.
theorem dirichlet_unit_right_below_step_no(h: Nat -> Nat, n: Nat, k: Nat) {
    not k.suc.divides(n) and sum(map(divisors_up_to(n, k), h)) = Nat.0
        implies sum(map(divisors_up_to(n, k.suc), h)) = Nat.0
} by {
    if not k.suc.divides(n) and sum(map(divisors_up_to(n, k), h)) = Nat.0 {
        divisors_up_to_suc_no(n, k)
        divisors_up_to(n, k.suc) = divisors_up_to(n, k)
        map(divisors_up_to(n, k.suc), h) = map(divisors_up_to(n, k), h)
        sum(map(divisors_up_to(n, k.suc), h)) = sum(map(divisors_up_to(n, k), h))
    }
}

/// Step case for the right-identity helper.
theorem dirichlet_unit_right_below_step(f: Nat -> Nat, n: Nat, k: Nat) {
    dirichlet_unit_right_below_pred(f, n, k)
        implies dirichlet_unit_right_below_pred(f, n, k.suc)
} by {
    if dirichlet_unit_right_below_pred(f, n, k) {
        if k.suc < n {
            k < k.suc
            k < n
            sum(map(divisors_up_to(n, k),
                dirichlet_term(f, nat_dirichlet_unit_fn, n))) = Nat.0
            if k.suc.divides(n) {
                dirichlet_unit_right_below_step_yes(f, n, k)
                sum(map(divisors_up_to(n, k.suc),
                    dirichlet_term(f, nat_dirichlet_unit_fn, n))) = Nat.0
            } else {
                dirichlet_unit_right_below_step_no(
                    dirichlet_term(f, nat_dirichlet_unit_fn, n), n, k)
                sum(map(divisors_up_to(n, k.suc),
                    dirichlet_term(f, nat_dirichlet_unit_fn, n))) = Nat.0
            }
            sum(map(divisors_up_to(n, k.suc),
                dirichlet_term(f, nat_dirichlet_unit_fn, n))) = Nat.0
        }
        dirichlet_unit_right_below_pred(f, n, k.suc)
    }
}

/// At any bound below `n`, the dirichlet-term sum with the Dirichlet unit is zero.
theorem dirichlet_unit_right_below(f: Nat -> Nat, n: Nat, k: Nat) {
    dirichlet_unit_right_below_pred(f, n, k)
} by {
    define p(x: Nat) -> Bool {
        dirichlet_unit_right_below_pred(f, n, x)
    }
    dirichlet_unit_right_below_base(f, n)
    p(Nat.0)
    forall(j: Nat) {
        if p(j) {
            dirichlet_unit_right_below_pred(f, n, j)
            dirichlet_unit_right_below_step(f, n, j)
            dirichlet_unit_right_below_pred(f, n, j.suc)
            p(j.suc)
        }
    }
    p(k)
}

/// The Dirichlet unit is a right identity for Dirichlet convolution on positive
/// arguments: `(f * unit)(n) = f(n)` for `n > 0`.
theorem dirichlet_convolve_unit_right(f: Nat -> Nat, n: Nat) {
    Nat.0 < n implies dirichlet_convolve(f, nat_dirichlet_unit_fn)(n) = f(n)
} by {
    if Nat.0 < n {
        let m: Nat satisfy { m.suc = n }
        m < m.suc
        m < n
        let h: Nat -> Nat = dirichlet_term(f, nat_dirichlet_unit_fn, n)
        divides_self(n)
        n.divides(n)
        m.suc = n
        m.suc.divides(n)
        divisors_up_to_suc_yes(n, m)
        divisors_up_to(n, m.suc) = List.cons(m.suc, divisors_up_to(n, m))
        divisor_list(n) = divisors_up_to(n, n)
        divisor_list(n) = List.cons(n, divisors_up_to(n, m))
        divisor_quotient_self(n)
        divisor_quotient(n, n) = Nat.1
        nat_dirichlet_unit_fn_at_one
        nat_dirichlet_unit_fn(Nat.1) = Nat.1
        h(n) = f(n) * nat_dirichlet_unit_fn(divisor_quotient(n, n))
        h(n) = f(n) * nat_dirichlet_unit_fn(Nat.1)
        h(n) = f(n) * Nat.1
        f(n) * Nat.1 = f(n)
        h(n) = f(n)
        dirichlet_unit_right_below(f, n, m)
        dirichlet_unit_right_below_pred(f, n, m)
        sum(map(divisors_up_to(n, m), dirichlet_term(f, nat_dirichlet_unit_fn, n))) = Nat.0
        map(divisors_up_to(n, m), h) =
            map(divisors_up_to(n, m), dirichlet_term(f, nat_dirichlet_unit_fn, n))
        sum(map(divisors_up_to(n, m), h)) = Nat.0
        map(List.cons(n, divisors_up_to(n, m)), h) =
            List.cons(h(n), map(divisors_up_to(n, m), h))
        map(divisor_list(n), h) = List.cons(f(n), map(divisors_up_to(n, m), h))
        sum(List.cons(f(n), map(divisors_up_to(n, m), h))) =
            f(n) + sum(map(divisors_up_to(n, m), h))
        sum(map(divisor_list(n), h)) = f(n) + Nat.0
        f(n) + Nat.0 = f(n)
        sum(map(divisor_list(n), h)) = f(n)
        dirichlet_convolve(f, nat_dirichlet_unit_fn)(n) =
            sum(map(divisor_list(n), dirichlet_term(f, nat_dirichlet_unit_fn, n)))
        dirichlet_convolve(f, nat_dirichlet_unit_fn)(n) = f(n)
    }
}

/// At any non-one argument `d != 1`, the dirichlet term with the Dirichlet unit
/// on the left vanishes.
theorem dirichlet_term_unit_left_off_one(f: Nat -> Nat, n: Nat, d: Nat) {
    d != Nat.1
        implies dirichlet_term(nat_dirichlet_unit_fn, f, n)(d) = Nat.0
} by {
    if d != Nat.1 {
        nat_dirichlet_unit_fn_off_one(d)
        nat_dirichlet_unit_fn(d) = Nat.0
        dirichlet_term_apply(nat_dirichlet_unit_fn, f, n, d)
        dirichlet_term(nat_dirichlet_unit_fn, f, n)(d) =
            nat_dirichlet_unit_fn(d) * f(divisor_quotient(n, d))
        nat_dirichlet_unit_fn(d) * f(divisor_quotient(n, d)) =
            Nat.0 * f(divisor_quotient(n, d))
        Nat.0 * f(divisor_quotient(n, d)) = Nat.0
        dirichlet_term(nat_dirichlet_unit_fn, f, n)(d) = Nat.0
    }
}

/// Inductive predicate for the left-identity helper: at any bound `k >= 1`,
/// the dirichlet-term sum with the Dirichlet unit on the left equals `f(n)`,
/// because only the divisor `d = 1` contributes.
define dirichlet_unit_left_below_pred(f: Nat -> Nat, n: Nat, k: Nat) -> Bool {
    Nat.1 <= k implies
        sum(map(divisors_up_to(n, k),
            dirichlet_term(nat_dirichlet_unit_fn, f, n))) = f(n)
}

/// Base case for the left-identity helper (vacuous at `k = 0`).
theorem dirichlet_unit_left_below_base(f: Nat -> Nat, n: Nat) {
    dirichlet_unit_left_below_pred(f, n, Nat.0)
} by {
    if Nat.1 <= Nat.0 {
        false
    }
}

/// At bound `k.suc = 1` the divisor list is `[1]` and the dirichlet term there
/// equals `f(n)`.
theorem dirichlet_unit_left_below_step_one(f: Nat -> Nat, n: Nat) {
    sum(map(divisors_up_to(n, Nat.1),
        dirichlet_term(nat_dirichlet_unit_fn, f, n))) = f(n)
} by {
    divisors_up_to_one(n)
    divisors_up_to(n, Nat.1) = List.cons(Nat.1, List.nil[Nat])
    let h: Nat -> Nat = dirichlet_term(nat_dirichlet_unit_fn, f, n)
    divisor_quotient_one(n)
    divisor_quotient(n, Nat.1) = n
    nat_dirichlet_unit_fn_at_one
    nat_dirichlet_unit_fn(Nat.1) = Nat.1
    h(Nat.1) = nat_dirichlet_unit_fn(Nat.1) * f(divisor_quotient(n, Nat.1))
    h(Nat.1) = Nat.1 * f(n)
    Nat.1 * f(n) = f(n)
    h(Nat.1) = f(n)
    map(List.cons(Nat.1, List.nil[Nat]), h) =
        List.cons(h(Nat.1), map(List.nil[Nat], h))
    map(List.nil[Nat], h) = List.nil[Nat]
    map(List.cons(Nat.1, List.nil[Nat]), h) = List.cons(f(n), List.nil[Nat])
    sum(List.cons(f(n), List.nil[Nat])) = f(n) + sum(List.nil[Nat])
    sum(List.nil[Nat]) = Nat.0
    f(n) + Nat.0 = f(n)
    sum(List.cons(f(n), List.nil[Nat])) = f(n)
    sum(map(List.cons(Nat.1, List.nil[Nat]), h)) = f(n)
    map(divisors_up_to(n, Nat.1), h) = map(List.cons(Nat.1, List.nil[Nat]), h)
}

/// Cons-step lemma for the left identity: a zero head value preserves the
/// tail sum.
theorem sum_cons_zero_head_eq(h: Nat -> Nat, head: Nat, tail: List[Nat], v: Nat) {
    h(head) = Nat.0 and sum(map(tail, h)) = v
        implies sum(map(List.cons(head, tail), h)) = v
} by {
    if h(head) = Nat.0 and sum(map(tail, h)) = v {
        map(List.cons(head, tail), h) = List.cons(h(head), map(tail, h))
        h(head) = Nat.0
        map(List.cons(head, tail), h) = List.cons(Nat.0, map(tail, h))
        sum(List.cons(Nat.0, map(tail, h))) = Nat.0 + sum(map(tail, h))
        sum(map(tail, h)) = v
        Nat.0 + v = v
        sum(map(List.cons(head, tail), h)) = v
    }
}

/// Step case for `k >= 1` with `k.suc` a divisor: the head contributes zero,
/// preserving the sum at `f(n)`.
theorem dirichlet_unit_left_below_step_above_yes(f: Nat -> Nat, n: Nat, k: Nat) {
    Nat.1 <= k and k.suc.divides(n) and
        sum(map(divisors_up_to(n, k),
            dirichlet_term(nat_dirichlet_unit_fn, f, n))) = f(n)
        implies sum(map(divisors_up_to(n, k.suc),
            dirichlet_term(nat_dirichlet_unit_fn, f, n))) = f(n)
} by {
    if Nat.1 <= k and k.suc.divides(n) and
            sum(map(divisors_up_to(n, k),
                dirichlet_term(nat_dirichlet_unit_fn, f, n))) = f(n) {
        if k.suc = Nat.1 {
            k = Nat.0
            not (Nat.1 <= Nat.0)
            false
        }
        k.suc != Nat.1
        dirichlet_term_unit_left_off_one(f, n, k.suc)
        dirichlet_term(nat_dirichlet_unit_fn, f, n)(k.suc) = Nat.0
        divisors_up_to_suc_yes(n, k)
        divisors_up_to(n, k.suc) = List.cons(k.suc, divisors_up_to(n, k))
        sum_cons_zero_head_eq(dirichlet_term(nat_dirichlet_unit_fn, f, n),
            k.suc, divisors_up_to(n, k), f(n))
        sum(map(List.cons(k.suc, divisors_up_to(n, k)),
            dirichlet_term(nat_dirichlet_unit_fn, f, n))) = f(n)
        map(divisors_up_to(n, k.suc), dirichlet_term(nat_dirichlet_unit_fn, f, n)) =
            map(List.cons(k.suc, divisors_up_to(n, k)),
                dirichlet_term(nat_dirichlet_unit_fn, f, n))
        sum(map(divisors_up_to(n, k.suc),
            dirichlet_term(nat_dirichlet_unit_fn, f, n))) = f(n)
    }
}

/// Step case for `k >= 1` with `k.suc` a non-divisor: the list is unchanged.
theorem dirichlet_unit_left_below_step_above_no(h: Nat -> Nat, n: Nat, k: Nat, v: Nat) {
    not k.suc.divides(n) and sum(map(divisors_up_to(n, k), h)) = v
        implies sum(map(divisors_up_to(n, k.suc), h)) = v
} by {
    if not k.suc.divides(n) and sum(map(divisors_up_to(n, k), h)) = v {
        divisors_up_to_suc_no(n, k)
        divisors_up_to(n, k.suc) = divisors_up_to(n, k)
        map(divisors_up_to(n, k.suc), h) = map(divisors_up_to(n, k), h)
        sum(map(divisors_up_to(n, k.suc), h)) = sum(map(divisors_up_to(n, k), h))
    }
}

/// Step case for `k >= 1`: the new bound `k.suc >= 2` contributes zero.
theorem dirichlet_unit_left_below_step_above(f: Nat -> Nat, n: Nat, k: Nat) {
    Nat.1 <= k and
        sum(map(divisors_up_to(n, k),
            dirichlet_term(nat_dirichlet_unit_fn, f, n))) = f(n)
        implies sum(map(divisors_up_to(n, k.suc),
            dirichlet_term(nat_dirichlet_unit_fn, f, n))) = f(n)
} by {
    if Nat.1 <= k and
            sum(map(divisors_up_to(n, k),
                dirichlet_term(nat_dirichlet_unit_fn, f, n))) = f(n) {
        if k.suc.divides(n) {
            dirichlet_unit_left_below_step_above_yes(f, n, k)
        } else {
            dirichlet_unit_left_below_step_above_no(
                dirichlet_term(nat_dirichlet_unit_fn, f, n), n, k, f(n))
        }
    }
}

/// Step case for the left-identity helper.
theorem dirichlet_unit_left_below_step(f: Nat -> Nat, n: Nat, k: Nat) {
    dirichlet_unit_left_below_pred(f, n, k)
        implies dirichlet_unit_left_below_pred(f, n, k.suc)
} by {
    if dirichlet_unit_left_below_pred(f, n, k) {
        if Nat.1 <= k.suc {
            if k = Nat.0 {
                Nat.0.suc = Nat.1
                k.suc = Nat.1
                dirichlet_unit_left_below_step_one(f, n)
                sum(map(divisors_up_to(n, Nat.1),
                    dirichlet_term(nat_dirichlet_unit_fn, f, n))) = f(n)
                sum(map(divisors_up_to(n, k.suc),
                    dirichlet_term(nat_dirichlet_unit_fn, f, n))) = f(n)
            } else {
                Nat.0 < k
                Nat.1 <= k
                sum(map(divisors_up_to(n, k),
                    dirichlet_term(nat_dirichlet_unit_fn, f, n))) = f(n)
                dirichlet_unit_left_below_step_above(f, n, k)
                sum(map(divisors_up_to(n, k.suc),
                    dirichlet_term(nat_dirichlet_unit_fn, f, n))) = f(n)
            }
            sum(map(divisors_up_to(n, k.suc),
                dirichlet_term(nat_dirichlet_unit_fn, f, n))) = f(n)
        }
        dirichlet_unit_left_below_pred(f, n, k.suc)
    }
}

/// At any bound `k`, the dirichlet-term sum with the Dirichlet unit on the left
/// equals `f(n)` whenever `k >= 1`.
theorem dirichlet_unit_left_below(f: Nat -> Nat, n: Nat, k: Nat) {
    dirichlet_unit_left_below_pred(f, n, k)
} by {
    define p(x: Nat) -> Bool {
        dirichlet_unit_left_below_pred(f, n, x)
    }
    dirichlet_unit_left_below_base(f, n)
    p(Nat.0)
    forall(j: Nat) {
        if p(j) {
            dirichlet_unit_left_below_pred(f, n, j)
            dirichlet_unit_left_below_step(f, n, j)
            dirichlet_unit_left_below_pred(f, n, j.suc)
            p(j.suc)
        }
    }
    p(k)
}

/// The Dirichlet unit is a left identity for Dirichlet convolution on positive
/// arguments: `(unit * f)(n) = f(n)` for `n > 0`.
theorem dirichlet_convolve_unit_left(f: Nat -> Nat, n: Nat) {
    Nat.0 < n implies dirichlet_convolve(nat_dirichlet_unit_fn, f)(n) = f(n)
} by {
    if Nat.0 < n {
        Nat.1 <= n
        dirichlet_unit_left_below(f, n, n)
        dirichlet_unit_left_below_pred(f, n, n)
        sum(map(divisors_up_to(n, n),
            dirichlet_term(nat_dirichlet_unit_fn, f, n))) = f(n)
        divisor_list(n) = divisors_up_to(n, n)
        map(divisor_list(n), dirichlet_term(nat_dirichlet_unit_fn, f, n)) =
            map(divisors_up_to(n, n), dirichlet_term(nat_dirichlet_unit_fn, f, n))
        sum(map(divisor_list(n), dirichlet_term(nat_dirichlet_unit_fn, f, n))) = f(n)
        dirichlet_convolve(nat_dirichlet_unit_fn, f)(n) =
            sum(map(divisor_list(n), dirichlet_term(nat_dirichlet_unit_fn, f, n)))
        dirichlet_convolve(nat_dirichlet_unit_fn, f)(n) = f(n)
    }
}

/// The Dirichlet convolution annihilates on the right by the constant-zero
/// arithmetic function.
theorem dirichlet_convolve_zero_right(f: Nat -> Nat) {
    dirichlet_convolve(f, nat_zero_arithmetic_fn) = nat_zero_arithmetic_fn
} by {
    forall(n: Nat) {
        let h: Nat -> Nat = dirichlet_term(f, nat_zero_arithmetic_fn, n)
        forall(d: Nat) {
            h(d) = f(d) * nat_zero_arithmetic_fn(divisor_quotient(n, d))
            nat_zero_arithmetic_fn(divisor_quotient(n, d)) = Nat.0
            h(d) = f(d) * Nat.0
            f(d) * Nat.0 = Nat.0
            h(d) = Nat.0
        }
        sum_map_zero_fn(divisor_list(n), h)
        sum(map(divisor_list(n), h)) = Nat.0
        dirichlet_convolve(f, nat_zero_arithmetic_fn)(n) =
            sum(map(divisor_list(n), dirichlet_term(f, nat_zero_arithmetic_fn, n)))
        dirichlet_convolve(f, nat_zero_arithmetic_fn)(n) = Nat.0
        nat_zero_arithmetic_fn(n) = Nat.0
        dirichlet_convolve(f, nat_zero_arithmetic_fn)(n) = nat_zero_arithmetic_fn(n)
    }
}

/// When `d` divides `n`, the cofactor `n / d` also divides `n`.
theorem divisor_quotient_divides(n: Nat, d: Nat) {
    d.divides(n) implies divisor_quotient(n, d).divides(n)
} by {
    if d.divides(n) {
        divisor_quotient_cofactor(n, d)
        d * divisor_quotient(n, d) = n
        divisor_quotient(n, d) * d = d * divisor_quotient(n, d)
        divisor_quotient(n, d) * d = n
        exists(c: Nat) { divisor_quotient(n, d) * c = n }
    }
}

/// For positive `n`, every divisor produces a positive cofactor.
theorem divisor_quotient_positive(n: Nat, d: Nat) {
    Nat.0 < n and d.divides(n) implies Nat.0 < divisor_quotient(n, d)
} by {
    if Nat.0 < n and d.divides(n) {
        divisor_quotient_cofactor(n, d)
        d * divisor_quotient(n, d) = n
        if divisor_quotient(n, d) = Nat.0 {
            d * Nat.0 = Nat.0
            n = Nat.0
            false
        }
        divisor_quotient(n, d) != Nat.0
        Nat.0 < divisor_quotient(n, d)
    }
}

/// For positive `n` every divisor `d` of `n` is itself positive.
theorem divisor_of_positive_is_positive(n: Nat, d: Nat) {
    Nat.0 < n and d.divides(n) implies Nat.0 < d
} by {
    if Nat.0 < n and d.divides(n) {
        let c: Nat satisfy { d * c = n }
        if d = Nat.0 {
            Nat.0 * c = Nat.0
            n = Nat.0
            false
        }
        d != Nat.0
        Nat.0 < d
    }
}

/// A product of divisors divides the product of the ambient values.
theorem divisor_product_divides(a: Nat, b: Nat, d: Nat, e: Nat) {
    d.divides(a) and e.divides(b) implies (d * e).divides(a * b)
} by {
    if d.divides(a) and e.divides(b) {
        let q: Nat satisfy { d * q = a }
        let r: Nat satisfy { e * r = b }
        (d * e) * (q * r) = (d * q) * (e * r)
        (d * q) * (e * r) = a * b
        (d * e) * (q * r) = a * b
        (d * e).divides(a * b) = exists(c: Nat) { (d * e) * c = a * b }
        exists(c: Nat) { (d * e) * c = a * b }
        (d * e).divides(a * b)
    }
}

/// Products of positive divisors appear in the divisor list of the product.
theorem divisor_product_in_divisor_list(a: Nat, b: Nat, d: Nat, e: Nat) {
    Nat.0 < a and Nat.0 < b and divisor_list(a).contains(d) and divisor_list(b).contains(e)
        implies divisor_list(a * b).contains(d * e)
} by {
    if Nat.0 < a and Nat.0 < b and divisor_list(a).contains(d) and divisor_list(b).contains(e) {
        divisor_list_contains_implies(a, d)
        Nat.0 < d and d.divides(a)
        Nat.0 < d
        d.divides(a)
        divisor_list_contains_implies(b, e)
        Nat.0 < e and e.divides(b)
        Nat.0 < e
        e.divides(b)
        divisor_product_divides(a, b, d, e)
        (d * e).divides(a * b)
        if d * e = Nat.0 {
            mul_to_zero(d, e)
            d = Nat.0 or e = Nat.0
            if d = Nat.0 {
                Nat.0 < Nat.0
                lt_not_ref(Nat.0)
                false
            }
            if e = Nat.0 {
                Nat.0 < Nat.0
                lt_not_ref(Nat.0)
                false
            }
            false
        }
        d * e != Nat.0
        Nat.0 < d * e
        if a * b = Nat.0 {
            mul_to_zero(a, b)
            a = Nat.0 or b = Nat.0
            if a = Nat.0 {
                Nat.0 < Nat.0
                lt_not_ref(Nat.0)
                false
            }
            if b = Nat.0 {
                Nat.0 < Nat.0
                lt_not_ref(Nat.0)
                false
            }
            false
        }
        a * b != Nat.0
        Nat.0 < a * b
        divisor_list_contains_of(a * b, d * e)
        divisor_list(a * b).contains(d * e)
    }
}

/// Cofactors multiply across products of divisors.
theorem divisor_quotient_product_reassoc(a: Nat, b: Nat, d: Nat, e: Nat) {
    Nat.0 < a and Nat.0 < b and d.divides(a) and e.divides(b)
        implies divisor_quotient(a * b, d * e) =
            divisor_quotient(a, d) * divisor_quotient(b, e)
} by {
    if Nat.0 < a and Nat.0 < b and d.divides(a) and e.divides(b) {
        let q: Nat = divisor_quotient(a, d)
        let r: Nat = divisor_quotient(b, e)
        divisor_quotient_cofactor(a, d)
        d * q = a
        divisor_quotient_cofactor(b, e)
        e * r = b
        (d * e) * (q * r) = (d * q) * (e * r)
        (d * q) * (e * r) = a * b
        (d * e) * (q * r) = a * b
        divisor_product_divides(a, b, d, e)
        (d * e).divides(a * b)
        divisor_quotient_cofactor(a * b, d * e)
        (d * e) * divisor_quotient(a * b, d * e) = a * b
        (d * e) * divisor_quotient(a * b, d * e) = (d * e) * (q * r)
        divisor_of_positive_is_positive(a, d)
        Nat.0 < d
        divisor_of_positive_is_positive(b, e)
        Nat.0 < e
        if d * e = Nat.0 {
            mul_to_zero(d, e)
            d = Nat.0 or e = Nat.0
            if d = Nat.0 {
                Nat.0 < Nat.0
                lt_not_ref(Nat.0)
                false
            }
            if e = Nat.0 {
                Nat.0 < Nat.0
                lt_not_ref(Nat.0)
                false
            }
            false
        }
        d * e != Nat.0
        mul_cancel_left(d * e, divisor_quotient(a * b, d * e), q * r)
        divisor_quotient(a * b, d * e) = q * r
        q = divisor_quotient(a, d)
        r = divisor_quotient(b, e)
        divisor_quotient(a * b, d * e) =
            divisor_quotient(a, d) * divisor_quotient(b, e)
    }
    Nat.0 < a and Nat.0 < b and d.divides(a) and e.divides(b) implies divisor_quotient(a * b, d * e) = divisor_quotient(a, d) * divisor_quotient(b, e)
}

/// True when a Dirichlet product-summand split may be used.
define dirichlet_term_mul_coprime_divisors_hyp(f: Nat -> Nat, g: Nat -> Nat,
    a: Nat, b: Nat, d: Nat, e: Nat) -> Bool {
    is_multiplicative_nat_fn(f) and is_multiplicative_nat_fn(g) and
    Nat.0 < a and Nat.0 < b and a.coprime(b) and d.divides(a) and e.divides(b)
}

/// On coprime ambient values, a product divisor's Dirichlet summand splits as
/// the product of the two separate summands.
theorem dirichlet_term_mul_of_coprime_divisors(f: Nat -> Nat, g: Nat -> Nat,
    a: Nat, b: Nat, d: Nat, e: Nat) {
    dirichlet_term_mul_coprime_divisors_hyp(f, g, a, b, d, e)
        implies dirichlet_term(f, g, a * b)(d * e) =
            dirichlet_term(f, g, a)(d) * dirichlet_term(f, g, b)(e)
} by {
    if dirichlet_term_mul_coprime_divisors_hyp(f, g, a, b, d, e) {
        dirichlet_term_mul_coprime_divisors_hyp(f, g, a, b, d, e) =
            (is_multiplicative_nat_fn(f) and is_multiplicative_nat_fn(g) and
                Nat.0 < a and Nat.0 < b and a.coprime(b) and d.divides(a) and e.divides(b))
        let q: Nat = divisor_quotient(a, d)
        let r: Nat = divisor_quotient(b, e)
        dirichlet_term_apply(f, g, a * b, d * e)
        dirichlet_term(f, g, a * b)(d * e) =
            f(d * e) * g(divisor_quotient(a * b, d * e))
        divisor_quotient_product_reassoc(a, b, d, e)
        divisor_quotient(a * b, d * e) =
            divisor_quotient(a, d) * divisor_quotient(b, e)
        q = divisor_quotient(a, d)
        r = divisor_quotient(b, e)
        divisor_quotient(a * b, d * e) =
            divisor_quotient(a, d) * divisor_quotient(b, e)
        divisor_quotient(a, d) * divisor_quotient(b, e) = divisor_quotient(a, d) * r
        divisor_quotient(a, d) * r = q * r
        divisor_quotient(a * b, d * e) = q * r

        coprime_of_divisors(a, b, d, e)
        (a.coprime(b) and d.divides(a) and e.divides(b)) implies d.coprime(e)
        d.coprime(e)
        if is_multiplicative_nat_fn(f) and d.coprime(e) {
            multiplicative_nat_fn_apply(f, d, e)
            f(d * e) = f(d) * f(e)
        }

        divisor_quotient_divides(a, d)
        d.divides(a) implies divisor_quotient(a, d).divides(a)
        q.divides(a)
        divisor_quotient_divides(b, e)
        e.divides(b) implies divisor_quotient(b, e).divides(b)
        r.divides(b)
        coprime_of_divisors(a, b, q, r)
        a.coprime(b) and q.divides(a) and r.divides(b) implies q.coprime(r)
        q.coprime(r)
        if is_multiplicative_nat_fn(g) and q.coprime(r) {
            multiplicative_nat_fn_apply(g, q, r)
            g(q * r) = g(q) * g(r)
        }

        dirichlet_term_apply(f, g, a, d)
        dirichlet_term(f, g, a)(d) = f(d) * g(q)
        dirichlet_term_apply(f, g, b, e)
        dirichlet_term(f, g, b)(e) = f(e) * g(r)
        nat_mul_swap_middle(f(d), f(e), g(q), g(r))
        (f(d) * f(e)) * (g(q) * g(r)) = (f(d) * g(q)) * (f(e) * g(r))
        dirichlet_term(f, g, a * b)(d * e) =
            (f(d) * g(q)) * (f(e) * g(r))
        dirichlet_term(f, g, a * b)(d * e) =
            dirichlet_term(f, g, a)(d) * dirichlet_term(f, g, b)(e)
    }
}

/// Multiply the two components of a pair of natural numbers.
define divisor_pair_mul(p: Pair[Nat, Nat]) -> Nat {
    p.first * p.second
}

/// Products of divisors chosen independently from `a` and `b`.
define divisor_pair_product_values(a: Nat, b: Nat) -> List[Nat] {
    map(list_pair_product(divisor_list(a), divisor_list(b)), divisor_pair_mul)
}

/// An explicit product of two listed divisors occurs in the paired product list.
theorem divisor_pair_product_values_contains_of_contains(a: Nat, b: Nat, d: Nat, e: Nat) {
    Nat.0 < a and Nat.0 < b and
    divisor_list(a).contains(d) and divisor_list(b).contains(e)
        implies divisor_pair_product_values(a, b).contains(d * e)
} by {
    if Nat.0 < a and Nat.0 < b and divisor_list(a).contains(d) and divisor_list(b).contains(e) {
        list_pair_product_contains_of_contains(divisor_list(a), divisor_list(b), d, e)
        list_pair_product(divisor_list(a), divisor_list(b)).contains(Pair.new(d, e))
        map_contains_of_contains(list_pair_product(divisor_list(a), divisor_list(b)),
            divisor_pair_mul, Pair.new(d, e))
        map(list_pair_product(divisor_list(a), divisor_list(b)), divisor_pair_mul).contains(
            divisor_pair_mul(Pair.new(d, e)))
        Pair.new(d, e).first = d
        Pair.new(d, e).second = e
        divisor_pair_mul(Pair.new(d, e)) = d * e
        divisor_pair_product_values(a, b).contains(d * e)
    }
}

/// Every value produced by multiplying a listed divisor of `a` by a listed divisor
/// of `b` is a divisor of `a * b`.
theorem divisor_pair_product_values_contained_in_divisor_list(a: Nat, b: Nat, x: Nat) {
    Nat.0 < a and Nat.0 < b and divisor_pair_product_values(a, b).contains(x)
        implies divisor_list(a * b).contains(x)
} by {
    if Nat.0 < a and Nat.0 < b and divisor_pair_product_values(a, b).contains(x) {
        divisor_pair_product_values(a, b) =
            map(list_pair_product(divisor_list(a), divisor_list(b)), divisor_pair_mul)
        map_contains(list_pair_product(divisor_list(a), divisor_list(b)), divisor_pair_mul, x)
        let p: Pair[Nat, Nat] satisfy {
            list_pair_product(divisor_list(a), divisor_list(b)).contains(p) and divisor_pair_mul(p) = x
        }
        list_pair_product_contains_imp_left(divisor_list(a), divisor_list(b), p)
        divisor_list(a).contains(p.first)
        list_pair_product_contains_imp_right(divisor_list(a), divisor_list(b), p)
        divisor_list(b).contains(p.second)
        divisor_product_in_divisor_list(a, b, p.first, p.second)
        divisor_list(a * b).contains(p.first * p.second)
        divisor_pair_mul(p) = p.first * p.second
        p.first * p.second = x
        divisor_list(a * b).contains(x)
    }
}


/// A divisor of a product of coprime numbers factors into a divisor of each side.
theorem coprime_product_divisor_factors(a: Nat, b: Nat, r: Nat) {
    Nat.0 < a and Nat.0 < b and a.coprime(b) and divisor_list(a * b).contains(r)
        implies exists(d: Nat, e: Nat) {
            divisor_list(a).contains(d) and divisor_list(b).contains(e) and r = d * e
        }
} by {
    if Nat.0 < a and Nat.0 < b and a.coprime(b) and divisor_list(a * b).contains(r) {
        divisor_list_contains_implies(a * b, r)
        Nat.0 < r and r.divides(a * b)
        Nat.0 < r
        r.divides(a * b)
        let d: Nat = r.gcd(a)
        gcd_divides_left(r, a)
        d.divides(r)
        gcd_divides_right(r, a)
        d.divides(a)
        divisor_of_positive_is_positive(a, d)
        Nat.0 < d
        divisor_list_contains_of(a, d)
        divisor_list(a).contains(d)
        let e: Nat = divisor_quotient(r, d)
        divisor_quotient_cofactor(r, d)
        d * e = r
        divisor_quotient_positive(r, d)
        Nat.0 < e
        divisor_quotient_cofactor(a, d)
        d * divisor_quotient(a, d) = a
        divisor_quotient(a, d) * d = a
        e * d = d * e
        e * d = r
        if d = Nat.0 {
            Nat.0 < Nat.0
            lt_not_ref(Nat.0)
            false
        }
        d != Nat.0
        r.gcd(a) = d
        r.gcd(a) != Nat.0
        e * r.gcd(a) = r
        divisor_quotient(a, d) * r.gcd(a) = a
        cofactor(r, a, e, divisor_quotient(a, d))
        e.gcd(divisor_quotient(a, d)) = Nat.1
        e.coprime(divisor_quotient(a, d))
        r.divides(a * b)
        let q: Nat satisfy { r * q = a * b }
        r * q = a * b
        d * e * q = a * b
        d * (e * q) = d * e * q
        d * (e * q) = a * b
        divisor_quotient_cofactor(a, d)
        d * divisor_quotient(a, d) = a
        d * (e * q) = d * (divisor_quotient(a, d) * b)
        mul_cancel_left(d, e * q, divisor_quotient(a, d) * b)
        e * q = divisor_quotient(a, d) * b
        e.divides(divisor_quotient(a, d) * b)
        coprime_divides_of_divides_mul(e, divisor_quotient(a, d), b)
        e.divides(b)
        divisor_list_contains_of(b, e)
        divisor_list(b).contains(e)
        r = d * e
        exists(d0: Nat, e0: Nat) {
            divisor_list(a).contains(d0) and divisor_list(b).contains(e0) and r = d0 * e0
        }
    }
}

/// A divisor from the left list is coprime to a divisor from the right list
/// when the ambient numbers are coprime.
theorem divisor_pair_cross_coprime(a: Nat, b: Nat, p: Pair[Nat, Nat], q: Pair[Nat, Nat]) {
    a.coprime(b)
        and list_pair_product(divisor_list(a), divisor_list(b)).contains(p)
        and list_pair_product(divisor_list(a), divisor_list(b)).contains(q)
        implies p.first.coprime(q.second)
} by {
    if a.coprime(b)
            and list_pair_product(divisor_list(a), divisor_list(b)).contains(p)
            and list_pair_product(divisor_list(a), divisor_list(b)).contains(q) {
        list_pair_product_contains_imp_left(divisor_list(a), divisor_list(b), p)
        divisor_list(a).contains(p.first)
        divisor_list_contains_implies(a, p.first)
        p.first.divides(a)
        list_pair_product_contains_imp_right(divisor_list(a), divisor_list(b), q)
        divisor_list(b).contains(q.second)
        divisor_list_contains_implies(b, q.second)
        q.second.divides(b)
        a.coprime(b) and p.first.divides(a) and q.second.divides(b)
        coprime_of_divisors(a, b, p.first, q.second)
        p.first.coprime(q.second)
    }
}

/// Equality of pair products gives a left-factor divisibility witness after
/// swapping the factors of the right-hand product.
theorem divisor_pair_mul_eq_imp_first_divides_swapped(p: Pair[Nat, Nat], q: Pair[Nat, Nat]) {
    divisor_pair_mul(p) = divisor_pair_mul(q) implies p.first.divides(q.second * q.first)
} by {
    if divisor_pair_mul(p) = divisor_pair_mul(q) {
        divisor_pair_mul(p) = p.first * p.second
        divisor_pair_mul(q) = q.first * q.second
        p.first * p.second = q.first * q.second
        q.first * q.second = q.second * q.first
        p.first * p.second = q.second * q.first
        p.first.divides(q.second * q.first) =
            exists(c: Nat) { p.first * c = q.second * q.first }
        exists(c: Nat) { p.first * c = q.second * q.first }
        p.first.divides(q.second * q.first)
    }
}

/// Multiplication is injective on pairs of divisors of coprime natural numbers.
theorem divisor_pair_mul_injective_on_coprime_divisors(
        a: Nat, b: Nat, p: Pair[Nat, Nat], q: Pair[Nat, Nat]) {
    a.coprime(b)
        and list_pair_product(divisor_list(a), divisor_list(b)).contains(p)
        and list_pair_product(divisor_list(a), divisor_list(b)).contains(q)
        and divisor_pair_mul(p) = divisor_pair_mul(q)
        implies p = q
} by {
    if a.coprime(b)
            and list_pair_product(divisor_list(a), divisor_list(b)).contains(p)
            and list_pair_product(divisor_list(a), divisor_list(b)).contains(q)
            and divisor_pair_mul(p) = divisor_pair_mul(q) {
        list_pair_product_contains_imp_left(divisor_list(a), divisor_list(b), p)
        divisor_list(a).contains(p.first)
        list_pair_product_contains_imp_right(divisor_list(a), divisor_list(b), p)
        divisor_list(b).contains(p.second)
        list_pair_product_contains_imp_left(divisor_list(a), divisor_list(b), q)
        divisor_list(a).contains(q.first)
        list_pair_product_contains_imp_right(divisor_list(a), divisor_list(b), q)
        divisor_list(b).contains(q.second)

        divisor_list_contains_implies(a, p.first)
        Nat.0 < p.first and p.first.divides(a)
        Nat.0 < p.first
        p.first.divides(a)
        divisor_list_contains_implies(b, p.second)
        Nat.0 < p.second and p.second.divides(b)
        p.second.divides(b)
        divisor_list_contains_implies(a, q.first)
        Nat.0 < q.first and q.first.divides(a)
        q.first.divides(a)
        divisor_list_contains_implies(b, q.second)
        Nat.0 < q.second and q.second.divides(b)
        q.second.divides(b)

        divisor_pair_mul(p) = p.first * p.second
        divisor_pair_mul(q) = q.first * q.second
        p.first * p.second = q.first * q.second

        divisor_pair_cross_coprime(a, b, p, q)
        p.first.coprime(q.second)
        divisor_pair_mul_eq_imp_first_divides_swapped(p, q)
        p.first.divides(q.second * q.first)
        coprime_divides_of_divides_mul(p.first, q.second, q.first)
        p.first.divides(q.first)

        a.coprime(b) and
            list_pair_product(divisor_list(a), divisor_list(b)).contains(q) and
            list_pair_product(divisor_list(a), divisor_list(b)).contains(p)
        divisor_pair_cross_coprime(a, b, q, p)
        q.first.coprime(p.second)
        divisor_pair_mul(q) = divisor_pair_mul(p)
        divisor_pair_mul_eq_imp_first_divides_swapped(q, p)
        q.first.divides(p.second * p.first)
        coprime_divides_of_divides_mul(q.first, p.second, p.first)
        q.first.divides(p.first)

        divides_symm(p.first, q.first)
        p.first = q.first
        p.first * p.second = p.first * q.second
        p.first != Nat.0
        mul_cancel_left(p.first, p.second, q.second)
        p.second = q.second

        pair_eta(p)
        pair_eta(q)
        Pair.new(p.first, p.second) = p
        Pair.new(q.first, q.second) = q
        Pair.new(p.first, p.second) = Pair.new(q.first, q.second)
        p = q
    }
}

/// Products of divisors of coprime natural numbers are listed without duplicates.
theorem divisor_pair_product_values_is_unique(a: Nat, b: Nat) {
    a.coprime(b) implies divisor_pair_product_values(a, b).is_unique
} by {
    if a.coprime(b) {
        let items: List[Pair[Nat, Nat]] =
            list_pair_product(divisor_list(a), divisor_list(b))
        divisor_list_is_unique(a)
        divisor_list_is_unique(b)
        list_pair_product_unique(divisor_list(a), divisor_list(b))
        items.is_unique
        forall(p: Pair[Nat, Nat], q: Pair[Nat, Nat]) {
            if items.contains(p)
                    and items.contains(q)
                    and divisor_pair_mul(p) = divisor_pair_mul(q) {
                list_pair_product(divisor_list(a), divisor_list(b)).contains(p)
                list_pair_product(divisor_list(a), divisor_list(b)).contains(q)
                divisor_pair_mul_injective_on_coprime_divisors(a, b, p, q)
                p = q
            }
        }
        items.is_unique and forall(p: Pair[Nat, Nat], q: Pair[Nat, Nat]) {
            items.contains(p) and items.contains(q) and divisor_pair_mul(p) = divisor_pair_mul(q)
                implies p = q
        }
        locally_injective_map_is_unique[Pair[Nat, Nat], Nat](items, divisor_pair_mul)
        map[Pair[Nat, Nat], Nat](items, divisor_pair_mul).is_unique
        items = list_pair_product(divisor_list(a), divisor_list(b))
        divisor_pair_product_values(a, b) =
            map(list_pair_product(divisor_list(a), divisor_list(b)), divisor_pair_mul)
        divisor_pair_product_values(a, b).is_unique
    }
}

/// For positive coprime arguments, the product values and the divisors of the
/// product contain exactly the same natural numbers.
theorem divisor_pair_product_values_same_contains(a: Nat, b: Nat, x: Nat) {
    Nat.0 < a and Nat.0 < b and a.coprime(b) implies
        divisor_pair_product_values(a, b).contains(x) = divisor_list(a * b).contains(x)
} by {
    if Nat.0 < a and Nat.0 < b and a.coprime(b) {
        if divisor_pair_product_values(a, b).contains(x) {
            divisor_pair_product_values_contained_in_divisor_list(a, b, x)
            divisor_list(a * b).contains(x)
        }
        if divisor_list(a * b).contains(x) {
            coprime_product_divisor_factors(a, b, x)
            let (d: Nat, e: Nat) satisfy {
                divisor_list(a).contains(d) and divisor_list(b).contains(e) and x = d * e
            }
            divisor_pair_product_values_contains_of_contains(a, b, d, e)
            divisor_pair_product_values(a, b).contains(d * e)
            divisor_pair_product_values(a, b).contains(x)
        }
        divisor_pair_product_values(a, b).contains(x) = divisor_list(a * b).contains(x)
    }
}

/// For positive coprime arguments, product values of divisor pairs permute the
/// divisors of the product.
theorem divisor_pair_product_values_permutation_divisor_list(a: Nat, b: Nat) {
    Nat.0 < a and Nat.0 < b and a.coprime(b) implies
        is_permutation(divisor_pair_product_values(a, b), divisor_list(a * b))
} by {
    if Nat.0 < a and Nat.0 < b and a.coprime(b) {
        divisor_pair_product_values_is_unique(a, b)
        divisor_pair_product_values(a, b).is_unique
        divisor_list_is_unique(a * b)
        divisor_list(a * b).is_unique
        forall(x: Nat) {
            divisor_pair_product_values_same_contains(a, b, x)
            divisor_pair_product_values(a, b).contains(x) = divisor_list(a * b).contains(x)
        }
        unique_same_contains_imp_permutation(divisor_pair_product_values(a, b), divisor_list(a * b))
        is_permutation(divisor_pair_product_values(a, b), divisor_list(a * b))
    }
}

/// True when the pair-product Dirichlet convolution sum may be factored.
define dirichlet_convolve_pair_product_sum_hyp(f: Nat -> Nat, g: Nat -> Nat,
    a: Nat, b: Nat) -> Bool {
    is_multiplicative_nat_fn(f) and is_multiplicative_nat_fn(g) and
    Nat.0 < a and Nat.0 < b and a.coprime(b)
}

/// The Dirichlet summand sum over divisor products factors as the product of
/// the two separate Dirichlet convolution values.
theorem dirichlet_convolve_pair_product_sum(f: Nat -> Nat, g: Nat -> Nat,
    a: Nat, b: Nat) {
    dirichlet_convolve_pair_product_sum_hyp(f, g, a, b) implies
        sum(map(divisor_pair_product_values(a, b), dirichlet_term(f, g, a * b))) =
            dirichlet_convolve(f, g)(a) * dirichlet_convolve(f, g)(b)
} by {
    if dirichlet_convolve_pair_product_sum_hyp(f, g, a, b) {
        dirichlet_convolve_pair_product_sum_hyp(f, g, a, b) =
            (is_multiplicative_nat_fn(f) and is_multiplicative_nat_fn(g) and
                Nat.0 < a and Nat.0 < b and a.coprime(b))
        let items: List[Pair[Nat, Nat]] =
            list_pair_product(divisor_list(a), divisor_list(b))
        let term_ab: Nat -> Nat = dirichlet_term(f, g, a * b)
        let pair_term: Pair[Nat, Nat] -> Nat =
            pair_product_map_value(dirichlet_term(f, g, a), dirichlet_term(f, g, b))
        let lifted_term: Pair[Nat, Nat] -> Nat = compose(term_ab, divisor_pair_mul)

        divisor_pair_product_values(a, b) = map(items, divisor_pair_mul)
        map_map[Pair[Nat, Nat], Nat, Nat](items, divisor_pair_mul, term_ab)
        map(map(items, divisor_pair_mul), term_ab) =
            map(items, compose(term_ab, divisor_pair_mul))
        map(divisor_pair_product_values(a, b), term_ab) = map(items, lifted_term)

        forall(p: Pair[Nat, Nat]) {
            if items.contains(p) {
                list_pair_product_contains_imp_left(divisor_list(a), divisor_list(b), p)
                divisor_list(a).contains(p.first)
                divisor_list_contains_implies(a, p.first)
                Nat.0 < p.first and p.first.divides(a)
                p.first.divides(a)
                list_pair_product_contains_imp_right(divisor_list(a), divisor_list(b), p)
                divisor_list(b).contains(p.second)
                divisor_list_contains_implies(b, p.second)
                Nat.0 < p.second and p.second.divides(b)
                p.second.divides(b)

                dirichlet_term_mul_coprime_divisors_hyp(f, g, a, b, p.first, p.second) =
                    (is_multiplicative_nat_fn(f) and is_multiplicative_nat_fn(g) and
                        Nat.0 < a and Nat.0 < b and a.coprime(b) and
                        p.first.divides(a) and p.second.divides(b))
                dirichlet_term_mul_coprime_divisors_hyp(f, g, a, b, p.first, p.second)
                dirichlet_term_mul_of_coprime_divisors(f, g, a, b, p.first, p.second)
                dirichlet_term(f, g, a * b)(p.first * p.second) =
                    dirichlet_term(f, g, a)(p.first) * dirichlet_term(f, g, b)(p.second)

                lifted_term(p) = term_ab(divisor_pair_mul(p))
                divisor_pair_mul(p) = p.first * p.second
                term_ab(divisor_pair_mul(p)) =
                    dirichlet_term(f, g, a * b)(p.first * p.second)
                pair_term(p) =
                    dirichlet_term(f, g, a)(p.first) * dirichlet_term(f, g, b)(p.second)
                lifted_term(p) = pair_term(p)
            }
        }
        sum_map_of_pointwise[Pair[Nat, Nat], Nat](items, lifted_term, pair_term)
        sum(map(items, lifted_term)) = sum(map(items, pair_term))
        sum(map(divisor_pair_product_values(a, b), term_ab)) = sum(map(items, pair_term))

        list_pair_product_map_mul_sum[Nat, Nat](
            divisor_list(a), divisor_list(b), dirichlet_term(f, g, a), dirichlet_term(f, g, b))
        sum(map(items, pair_term)) =
            sum(map(divisor_list(a), dirichlet_term(f, g, a))) *
                sum(map(divisor_list(b), dirichlet_term(f, g, b)))

        dirichlet_convolve_apply(f, g, a)
        dirichlet_convolve(f, g)(a) = sum(map(divisor_list(a), dirichlet_term(f, g, a)))
        dirichlet_convolve_apply(f, g, b)
        dirichlet_convolve(f, g)(b) = sum(map(divisor_list(b), dirichlet_term(f, g, b)))
        sum(map(divisor_pair_product_values(a, b), dirichlet_term(f, g, a * b))) =
            dirichlet_convolve(f, g)(a) * dirichlet_convolve(f, g)(b)
    }
}

/// Dirichlet convolution of multiplicative functions is multiplicative on
/// positive coprime arguments.
theorem dirichlet_convolve_mul_coprime_positive(f: Nat -> Nat, g: Nat -> Nat,
    a: Nat, b: Nat) {
    dirichlet_convolve_pair_product_sum_hyp(f, g, a, b) implies
        dirichlet_convolve(f, g)(a * b) =
            dirichlet_convolve(f, g)(a) * dirichlet_convolve(f, g)(b)
} by {
    if dirichlet_convolve_pair_product_sum_hyp(f, g, a, b) {
        dirichlet_convolve_pair_product_sum_hyp(f, g, a, b) =
            (is_multiplicative_nat_fn(f) and is_multiplicative_nat_fn(g) and
                Nat.0 < a and Nat.0 < b and a.coprime(b))
        let term_ab: Nat -> Nat = dirichlet_term(f, g, a * b)
        divisor_list_is_unique(a * b)
        divisor_list(a * b).is_unique
        divisor_pair_product_values_is_unique(a, b)
        divisor_pair_product_values(a, b).is_unique
        forall(x: Nat) {
            divisor_pair_product_values_same_contains(a, b, x)
            divisor_pair_product_values(a, b).contains(x) = divisor_list(a * b).contains(x)
            divisor_list(a * b).contains(x) = divisor_pair_product_values(a, b).contains(x)
        }
        unique_same_contains_map_sum_eq[Nat, Nat](
            divisor_list(a * b), divisor_pair_product_values(a, b), term_ab)
        sum(map(divisor_list(a * b), term_ab)) =
            sum(map(divisor_pair_product_values(a, b), term_ab))

        dirichlet_convolve_pair_product_sum(f, g, a, b)
        sum(map(divisor_pair_product_values(a, b), term_ab)) =
            dirichlet_convolve(f, g)(a) * dirichlet_convolve(f, g)(b)

        dirichlet_convolve_apply(f, g, a * b)
        dirichlet_convolve(f, g)(a * b) = sum(map(divisor_list(a * b), term_ab))
        dirichlet_convolve(f, g)(a * b) =
            dirichlet_convolve(f, g)(a) * dirichlet_convolve(f, g)(b)
    }
}

/// Dirichlet convolution preserves multiplicative arithmetic functions.
theorem dirichlet_convolve_multiplicative(f: Nat -> Nat, g: Nat -> Nat) {
    is_multiplicative_nat_fn(f) and is_multiplicative_nat_fn(g)
        implies is_multiplicative_nat_fn(dirichlet_convolve(f, g))
} by {
    if is_multiplicative_nat_fn(f) and is_multiplicative_nat_fn(g) {
        is_multiplicative_nat_fn_at_one(f)
        f(Nat.1) = Nat.1
        is_multiplicative_nat_fn_at_one(g)
        g(Nat.1) = Nat.1
        dirichlet_convolve_at_one(f, g)
        dirichlet_convolve(f, g)(Nat.1) = f(Nat.1) * g(Nat.1)
        dirichlet_convolve(f, g)(Nat.1) = Nat.1

        forall(a: Nat, b: Nat) {
            if a.coprime(b) {
                if a = Nat.0 {
                    coprime_zero_left_imp_one(b)
                    b = Nat.1
                    a * b = Nat.0
                    dirichlet_convolve_at_zero(f, g)
                    dirichlet_convolve(f, g)(a * b) = Nat.0
                    dirichlet_convolve(f, g)(a) = Nat.0
                    dirichlet_convolve(f, g)(b) = Nat.1
                    dirichlet_convolve(f, g)(a) * dirichlet_convolve(f, g)(b) = Nat.0
                    dirichlet_convolve(f, g)(a * b) =
                        dirichlet_convolve(f, g)(a) * dirichlet_convolve(f, g)(b)
                } else {
                    if b = Nat.0 {
                        coprime_zero_right_imp_one(a)
                        a = Nat.1
                        a * b = Nat.0
                        dirichlet_convolve_at_zero(f, g)
                        dirichlet_convolve(f, g)(a * b) = Nat.0
                        dirichlet_convolve(f, g)(a) = Nat.1
                        dirichlet_convolve(f, g)(b) = Nat.0
                        dirichlet_convolve(f, g)(a) * dirichlet_convolve(f, g)(b) = Nat.0
                        dirichlet_convolve(f, g)(a * b) =
                            dirichlet_convolve(f, g)(a) * dirichlet_convolve(f, g)(b)
                    } else {
                        Nat.0 < a
                        Nat.0 < b
                        dirichlet_convolve_pair_product_sum_hyp(f, g, a, b) =
                            (is_multiplicative_nat_fn(f) and is_multiplicative_nat_fn(g) and
                                Nat.0 < a and Nat.0 < b and a.coprime(b))
                        dirichlet_convolve_pair_product_sum_hyp(f, g, a, b)
                        dirichlet_convolve_mul_coprime_positive(f, g, a, b)
                        dirichlet_convolve(f, g)(a * b) =
                            dirichlet_convolve(f, g)(a) * dirichlet_convolve(f, g)(b)
                    }
                }
            }
        }
        is_multiplicative_nat_fn(dirichlet_convolve(f, g))
    }
}

/// The number of divisors is multiplicative on positive coprime arguments.
theorem nat_tau_mul_coprime_positive(a: Nat, b: Nat) {
    Nat.0 < a and Nat.0 < b and a.coprime(b) implies
        nat_tau(a * b) = nat_tau(a) * nat_tau(b)
} by {
    if Nat.0 < a and Nat.0 < b and a.coprime(b) {
        divisor_pair_product_values_permutation_divisor_list(a, b)
        is_permutation(divisor_pair_product_values(a, b), divisor_list(a * b))
        permutation_preserves_length(
            divisor_pair_product_values(a, b),
            divisor_list(a * b)
        )
        divisor_pair_product_values(a, b).length = divisor_list(a * b).length

        let items: List[Pair[Nat, Nat]] =
            list_pair_product(divisor_list(a), divisor_list(b))
        divisor_pair_product_values(a, b) = map(items, divisor_pair_mul)
        map_length[Pair[Nat, Nat], Nat](items, divisor_pair_mul)
        divisor_pair_product_values(a, b).length = items.length
        list_pair_product_length(divisor_list(a), divisor_list(b))
        items.length = divisor_list(b).length * divisor_list(a).length
        divisor_list(b).length * divisor_list(a).length =
            divisor_list(a).length * divisor_list(b).length
        divisor_list(a * b).length = divisor_list(a).length * divisor_list(b).length
        nat_tau(a * b) = divisor_list(a * b).length
        nat_tau(a) = divisor_list(a).length
        nat_tau(b) = divisor_list(b).length
        nat_tau(a * b) = nat_tau(a) * nat_tau(b)
    }
}

/// The number of divisors is multiplicative on coprime arguments.
theorem nat_tau_multiplicative {
    is_multiplicative_nat_fn(nat_tau)
} by {
    nat_tau_one
    nat_tau(Nat.1) = Nat.1
    forall(a: Nat, b: Nat) {
        if a.coprime(b) {
            if a = Nat.0 {
                coprime_zero_left_imp_one(b)
                b = Nat.1
                a * b = Nat.0
                nat_tau_zero
                nat_tau(a * b) = Nat.0
                nat_tau(a) = Nat.0
                nat_tau(b) = Nat.1
                nat_tau(a) * nat_tau(b) = Nat.0
                nat_tau(a * b) = nat_tau(a) * nat_tau(b)
            } else {
                if b = Nat.0 {
                    coprime_zero_right_imp_one(a)
                    a = Nat.1
                    a * b = Nat.0
                    nat_tau_zero
                    nat_tau(a * b) = Nat.0
                    nat_tau(a) = Nat.1
                    nat_tau(b) = Nat.0
                    nat_tau(a) * nat_tau(b) = Nat.0
                    nat_tau(a * b) = nat_tau(a) * nat_tau(b)
                } else {
                    Nat.0 < a
                    Nat.0 < b
                    nat_tau_mul_coprime_positive(a, b)
                    nat_tau(a * b) = nat_tau(a) * nat_tau(b)
                }
            }
        }
    }
    is_multiplicative_nat_fn(nat_tau)
}

/// Summing the pair product over one fixed-left row multiplies by that left
/// entry.
theorem divisor_pair_mul_row_sum(x: Nat, right: List[Nat]) {
    sum(map(list_pair_with_left(x, right), divisor_pair_mul)) = x * sum(right)
} by {
    list_pair_with_left(x, right) = map(right, pair_with_left_fn[Nat, Nat](x))
    map_map[Nat, Pair[Nat, Nat], Nat](right, pair_with_left_fn[Nat, Nat](x), divisor_pair_mul)
    map(map(right, pair_with_left_fn[Nat, Nat](x)), divisor_pair_mul) =
        map(right, compose(divisor_pair_mul, pair_with_left_fn[Nat, Nat](x)))
    forall(y: Nat) {
        compose(divisor_pair_mul, pair_with_left_fn[Nat, Nat](x), y) =
            divisor_pair_mul(pair_with_left_fn[Nat, Nat](x, y))
        pair_with_left_fn[Nat, Nat](x, y) = Pair.new(x, y)
        divisor_pair_mul(Pair.new(x, y)) = x * y
        scalar_mul(x, y) = x * y
        compose(divisor_pair_mul, pair_with_left_fn[Nat, Nat](x), y) = scalar_mul(x, y)
    }
    compose(divisor_pair_mul, pair_with_left_fn[Nat, Nat](x)) = scalar_mul(x)
    map(right, compose(divisor_pair_mul, pair_with_left_fn[Nat, Nat](x))) =
        map(right, scalar_mul(x))
    sum_scalar_mul(x, right)
    x * sum(right) = sum(map(right, scalar_mul(x)))
    sum(map(list_pair_with_left(x, right), divisor_pair_mul)) = x * sum(right)
}

/// Summing pair products over a list Cartesian product factors as the product
/// of the two coordinate sums.
theorem divisor_pair_product_mul_sum(left: List[Nat], right: List[Nat]) {
    sum(map(list_pair_product(left, right), divisor_pair_mul)) = sum(left) * sum(right)
} by {
    define p(items: List[Nat]) -> Bool {
        sum(map(list_pair_product(items, right), divisor_pair_mul)) =
            sum(items) * sum(right)
    }

    list_pair_product(List.nil[Nat], right) = List.nil[Pair[Nat, Nat]]
    map[Pair[Nat, Nat], Nat](List.nil[Pair[Nat, Nat]], divisor_pair_mul) =
        List.nil[Nat]
    sum[Nat](List.nil[Nat]) = Nat.0
    sum[Nat](List.nil[Nat]) * sum(right) = Nat.0
    p(List.nil[Nat])

    forall(head: Nat, tail: List[Nat]) {
        if p(tail) {
            let row: List[Pair[Nat, Nat]] = list_pair_with_left(head, right)
            let rest: List[Pair[Nat, Nat]] = list_pair_product(tail, right)
            list_pair_product(List.cons(head, tail), right) = row + rest
            map_add[Pair[Nat, Nat], Nat](row, rest, divisor_pair_mul)
            map(row + rest, divisor_pair_mul) =
                map(row, divisor_pair_mul) + map(rest, divisor_pair_mul)
            sum_add(map(row, divisor_pair_mul), map(rest, divisor_pair_mul))
            sum(map(row + rest, divisor_pair_mul)) =
                sum(map(row, divisor_pair_mul)) + sum(map(rest, divisor_pair_mul))
            divisor_pair_mul_row_sum(head, right)
            sum(map(row, divisor_pair_mul)) = head * sum(right)
            p(tail)
            sum(map(rest, divisor_pair_mul)) = sum(tail) * sum(right)
            sum(map(list_pair_product(List.cons(head, tail), right), divisor_pair_mul)) =
                head * sum(right) + sum(tail) * sum(right)
            distrib_right(head, sum(tail), sum(right))
            (head + sum(tail)) * sum(right) =
                head * sum(right) + sum(tail) * sum(right)
            sum(List.cons(head, tail)) = head + sum(tail)
            sum(map(list_pair_product(List.cons(head, tail), right), divisor_pair_mul)) =
                sum(List.cons(head, tail)) * sum(right)
            p(List.cons(head, tail))
        }
    }
    List.induction(function(items: List[Nat]) { p(items) })
    p(left)
}

/// The sum of divisor-pair products factors as the product of divisor sums.
theorem divisor_pair_product_values_sum(a: Nat, b: Nat) {
    sum(divisor_pair_product_values(a, b)) = nat_sigma(a) * nat_sigma(b)
} by {
    let items: List[Pair[Nat, Nat]] =
        list_pair_product(divisor_list(a), divisor_list(b))
    divisor_pair_product_values(a, b) = map(items, divisor_pair_mul)
    divisor_pair_product_mul_sum(divisor_list(a), divisor_list(b))
    sum(map(items, divisor_pair_mul)) = sum(divisor_list(a)) * sum(divisor_list(b))
    nat_sigma(a) = sum(divisor_list(a))
    nat_sigma(b) = sum(divisor_list(b))
    sum(divisor_pair_product_values(a, b)) = nat_sigma(a) * nat_sigma(b)
}

/// The sum of divisors is multiplicative on positive coprime arguments.
theorem nat_sigma_mul_coprime_positive(a: Nat, b: Nat) {
    Nat.0 < a and Nat.0 < b and a.coprime(b) implies
        nat_sigma(a * b) = nat_sigma(a) * nat_sigma(b)
} by {
    if Nat.0 < a and Nat.0 < b and a.coprime(b) {
        divisor_pair_product_values_is_unique(a, b)
        divisor_pair_product_values(a, b).is_unique
        divisor_list_is_unique(a * b)
        divisor_list(a * b).is_unique
        forall(x: Nat) {
            divisor_pair_product_values_same_contains(a, b, x)
            divisor_pair_product_values(a, b).contains(x) = divisor_list(a * b).contains(x)
        }
        unique_same_contains_map_sum_eq[Nat, Nat](
            divisor_pair_product_values(a, b),
            divisor_list(a * b),
            nat_identity_arithmetic_fn
        )
        sum(map(divisor_pair_product_values(a, b), nat_identity_arithmetic_fn)) =
            sum(map(divisor_list(a * b), nat_identity_arithmetic_fn))
        sum_map_nat_identity_arithmetic_fn_eq_sum(divisor_pair_product_values(a, b))
        sum(map(divisor_pair_product_values(a, b), nat_identity_arithmetic_fn)) =
            sum(divisor_pair_product_values(a, b))
        sum_map_nat_identity_arithmetic_fn_eq_sum(divisor_list(a * b))
        sum(map(divisor_list(a * b), nat_identity_arithmetic_fn)) = sum(divisor_list(a * b))
        sum(divisor_pair_product_values(a, b)) = sum(divisor_list(a * b))
        divisor_pair_product_values_sum(a, b)
        sum(divisor_pair_product_values(a, b)) = nat_sigma(a) * nat_sigma(b)
        nat_sigma(a * b) = sum(divisor_list(a * b))
        nat_sigma(a * b) = nat_sigma(a) * nat_sigma(b)
    }
}

/// The sum of divisors is multiplicative on coprime arguments.
theorem nat_sigma_multiplicative {
    is_multiplicative_nat_fn(nat_sigma)
} by {
    nat_sigma_one
    nat_sigma(Nat.1) = Nat.1
    forall(a: Nat, b: Nat) {
        if a.coprime(b) {
            if a = Nat.0 {
                coprime_zero_left_imp_one(b)
                b = Nat.1
                a * b = Nat.0
                nat_sigma_zero
                nat_sigma(a * b) = Nat.0
                nat_sigma(a) = Nat.0
                nat_sigma(b) = Nat.1
                nat_sigma(a) * nat_sigma(b) = Nat.0
                nat_sigma(a * b) = nat_sigma(a) * nat_sigma(b)
            } else {
                if b = Nat.0 {
                    coprime_zero_right_imp_one(a)
                    a = Nat.1
                    a * b = Nat.0
                    nat_sigma_zero
                    nat_sigma(a * b) = Nat.0
                    nat_sigma(a) = Nat.1
                    nat_sigma(b) = Nat.0
                    nat_sigma(a) * nat_sigma(b) = Nat.0
                    nat_sigma(a * b) = nat_sigma(a) * nat_sigma(b)
                } else {
                    Nat.0 < a
                    Nat.0 < b
                    nat_sigma_mul_coprime_positive(a, b)
                    nat_sigma(a * b) = nat_sigma(a) * nat_sigma(b)
                }
            }
        }
    }
    is_multiplicative_nat_fn(nat_sigma)
}

/// The cofactor map `d -> n / d` is an involution on divisors of positive `n`.
theorem divisor_quotient_involution(n: Nat, d: Nat) {
    Nat.0 < n and d.divides(n)
        implies divisor_quotient(n, divisor_quotient(n, d)) = d
} by {
    if Nat.0 < n and d.divides(n) {
        divisor_quotient_cofactor(n, d)
        d * divisor_quotient(n, d) = n
        divisor_quotient_divides(n, d)
        divisor_quotient(n, d).divides(n)
        divisor_quotient_cofactor(n, divisor_quotient(n, d))
        divisor_quotient(n, d) * divisor_quotient(n, divisor_quotient(n, d)) = n
        divisor_quotient(n, d) * d = d * divisor_quotient(n, d)
        divisor_quotient(n, d) * d = n
        divisor_quotient(n, d) * divisor_quotient(n, divisor_quotient(n, d)) =
            divisor_quotient(n, d) * d
        divisor_of_positive_is_positive(n, d)
        Nat.0 < d
        divisor_quotient_positive(n, d)
        Nat.0 < divisor_quotient(n, d)
        divisor_quotient(n, d) != Nat.0
        if divisor_quotient(n, divisor_quotient(n, d)) != d {
            // Cancel the positive factor.
            false
        }
    }
}

/// Reindexing identity: at a divisor `d` of positive `n`, swapping `f` and `g`
/// and replacing the index by the cofactor leaves the dirichlet term unchanged.
theorem dirichlet_term_flip(f: Nat -> Nat, g: Nat -> Nat, n: Nat, d: Nat) {
    Nat.0 < n and d.divides(n)
        implies dirichlet_term(f, g, n)(d) =
            dirichlet_term(g, f, n)(divisor_quotient(n, d))
} by {
    if Nat.0 < n and d.divides(n) {
        let q: Nat = divisor_quotient(n, d)
        dirichlet_term_apply(f, g, n, d)
        dirichlet_term(f, g, n)(d) = f(d) * g(q)
        divisor_quotient_divides(n, d)
        q.divides(n)
        dirichlet_term_apply(g, f, n, q)
        dirichlet_term(g, f, n)(q) = g(q) * f(divisor_quotient(n, q))
        divisor_quotient_involution(n, d)
        divisor_quotient(n, q) = d
        dirichlet_term(g, f, n)(q) = g(q) * f(d)
        f(d) * g(q) = g(q) * f(d)
        dirichlet_term(f, g, n)(d) = dirichlet_term(g, f, n)(q)
        q = divisor_quotient(n, d)
        dirichlet_term(f, g, n)(d) =
            dirichlet_term(g, f, n)(divisor_quotient(n, d))
    }
}

/// The cofactor-indexed Dirichlet term: at index `d`, the swapped-pair
/// Dirichlet term evaluated at the cofactor `n / d`.
define dirichlet_cofactor_term(f: Nat -> Nat, g: Nat -> Nat, n: Nat) -> (Nat -> Nat) {
    function(d: Nat) { dirichlet_term(g, f, n)(divisor_quotient(n, d)) }
}

/// Application of the cofactor-indexed Dirichlet term.
theorem dirichlet_cofactor_term_apply(f: Nat -> Nat, g: Nat -> Nat, n: Nat, d: Nat) {
    dirichlet_cofactor_term(f, g, n)(d) =
        dirichlet_term(g, f, n)(divisor_quotient(n, d))
}

/// On divisors `d` of positive `n`, the Dirichlet term agrees with the
/// cofactor-indexed swapped-pair Dirichlet term.
theorem dirichlet_term_eq_cofactor_term(f: Nat -> Nat, g: Nat -> Nat, n: Nat, d: Nat) {
    Nat.0 < n and d.divides(n) implies
        dirichlet_term(f, g, n)(d) = dirichlet_cofactor_term(f, g, n)(d)
} by {
    if Nat.0 < n and d.divides(n) {
        dirichlet_term_flip(f, g, n, d)
        dirichlet_cofactor_term_apply(f, g, n, d)
    }
}

/// Inductive predicate for the swap form: at any bound `k`, the
/// dirichlet-term sum over `divisors_up_to(n, k)` equals the
/// cofactor-indexed swapped-pair sum.
define dirichlet_swap_below_pred(f: Nat -> Nat, g: Nat -> Nat, n: Nat, k: Nat) -> Bool {
    Nat.0 < n implies
        sum(map(divisors_up_to(n, k), dirichlet_term(f, g, n))) =
        sum(map(divisors_up_to(n, k), dirichlet_cofactor_term(f, g, n)))
}

/// Base case for the swap form: at bound zero the divisor list is empty.
theorem dirichlet_swap_below_base(f: Nat -> Nat, g: Nat -> Nat, n: Nat) {
    dirichlet_swap_below_pred(f, g, n, Nat.0)
} by {
    divisors_up_to_zero(n)
    divisors_up_to(n, Nat.0) = List.nil[Nat]
    map(List.nil[Nat], dirichlet_term(f, g, n)) = List.nil[Nat]
    map(List.nil[Nat], dirichlet_cofactor_term(f, g, n)) = List.nil[Nat]
    sum(List.nil[Nat]) = Nat.0
    sum(map(divisors_up_to(n, Nat.0), dirichlet_term(f, g, n))) = Nat.0
    sum(map(divisors_up_to(n, Nat.0), dirichlet_cofactor_term(f, g, n))) = Nat.0
}

/// Step case for the swap form with `k.suc` a divisor of `n`: the head terms
/// agree by `dirichlet_term_eq_cofactor_term`, and the tail sums agree by
/// induction.
theorem dirichlet_swap_below_step_yes(f: Nat -> Nat, g: Nat -> Nat, n: Nat, k: Nat) {
    Nat.0 < n and k.suc.divides(n) and
        sum(map(divisors_up_to(n, k), dirichlet_term(f, g, n))) =
        sum(map(divisors_up_to(n, k), dirichlet_cofactor_term(f, g, n)))
        implies sum(map(divisors_up_to(n, k.suc), dirichlet_term(f, g, n))) =
            sum(map(divisors_up_to(n, k.suc), dirichlet_cofactor_term(f, g, n)))
} by {
    if Nat.0 < n and k.suc.divides(n) and
            sum(map(divisors_up_to(n, k), dirichlet_term(f, g, n))) =
            sum(map(divisors_up_to(n, k), dirichlet_cofactor_term(f, g, n))) {
        divisors_up_to_suc_yes(n, k)
        divisors_up_to(n, k.suc) = List.cons(k.suc, divisors_up_to(n, k))
        dirichlet_term_eq_cofactor_term(f, g, n, k.suc)
        dirichlet_term(f, g, n)(k.suc) = dirichlet_cofactor_term(f, g, n)(k.suc)
        map(List.cons(k.suc, divisors_up_to(n, k)), dirichlet_term(f, g, n)) =
            List.cons(dirichlet_term(f, g, n)(k.suc),
                map(divisors_up_to(n, k), dirichlet_term(f, g, n)))
        map(List.cons(k.suc, divisors_up_to(n, k)), dirichlet_cofactor_term(f, g, n)) =
            List.cons(dirichlet_cofactor_term(f, g, n)(k.suc),
                map(divisors_up_to(n, k), dirichlet_cofactor_term(f, g, n)))
        sum(List.cons(dirichlet_term(f, g, n)(k.suc),
                map(divisors_up_to(n, k), dirichlet_term(f, g, n)))) =
            dirichlet_term(f, g, n)(k.suc) +
                sum(map(divisors_up_to(n, k), dirichlet_term(f, g, n)))
        sum(List.cons(dirichlet_cofactor_term(f, g, n)(k.suc),
                map(divisors_up_to(n, k), dirichlet_cofactor_term(f, g, n)))) =
            dirichlet_cofactor_term(f, g, n)(k.suc) +
                sum(map(divisors_up_to(n, k), dirichlet_cofactor_term(f, g, n)))
        dirichlet_term(f, g, n)(k.suc) +
                sum(map(divisors_up_to(n, k), dirichlet_term(f, g, n))) =
            dirichlet_cofactor_term(f, g, n)(k.suc) +
                sum(map(divisors_up_to(n, k), dirichlet_cofactor_term(f, g, n)))
        sum(map(divisors_up_to(n, k.suc), dirichlet_term(f, g, n))) =
            sum(map(divisors_up_to(n, k.suc), dirichlet_cofactor_term(f, g, n)))
    }
}

/// Step case for the swap form with `k.suc` a non-divisor: lists are unchanged.
theorem dirichlet_swap_below_step_no(f: Nat -> Nat, g: Nat -> Nat, n: Nat, k: Nat) {
    not k.suc.divides(n) and
        sum(map(divisors_up_to(n, k), dirichlet_term(f, g, n))) =
        sum(map(divisors_up_to(n, k), dirichlet_cofactor_term(f, g, n)))
        implies sum(map(divisors_up_to(n, k.suc), dirichlet_term(f, g, n))) =
            sum(map(divisors_up_to(n, k.suc), dirichlet_cofactor_term(f, g, n)))
} by {
    if not k.suc.divides(n) and
            sum(map(divisors_up_to(n, k), dirichlet_term(f, g, n))) =
            sum(map(divisors_up_to(n, k), dirichlet_cofactor_term(f, g, n))) {
        divisors_up_to_suc_no(n, k)
        divisors_up_to(n, k.suc) = divisors_up_to(n, k)
        map(divisors_up_to(n, k.suc), dirichlet_term(f, g, n)) =
            map(divisors_up_to(n, k), dirichlet_term(f, g, n))
        map(divisors_up_to(n, k.suc), dirichlet_cofactor_term(f, g, n)) =
            map(divisors_up_to(n, k), dirichlet_cofactor_term(f, g, n))
    }
}

/// Step case for the swap form.
theorem dirichlet_swap_below_step(f: Nat -> Nat, g: Nat -> Nat, n: Nat, k: Nat) {
    dirichlet_swap_below_pred(f, g, n, k)
        implies dirichlet_swap_below_pred(f, g, n, k.suc)
} by {
    if dirichlet_swap_below_pred(f, g, n, k) {
        if Nat.0 < n {
            dirichlet_swap_below_pred(f, g, n, k) =
                (Nat.0 < n implies
                    sum(map(divisors_up_to(n, k), dirichlet_term(f, g, n))) =
                    sum(map(divisors_up_to(n, k), dirichlet_cofactor_term(f, g, n))))
            sum(map(divisors_up_to(n, k), dirichlet_term(f, g, n))) =
                sum(map(divisors_up_to(n, k), dirichlet_cofactor_term(f, g, n)))
            if k.suc.divides(n) {
                dirichlet_swap_below_step_yes(f, g, n, k)
            } else {
                dirichlet_swap_below_step_no(f, g, n, k)
            }
            sum(map(divisors_up_to(n, k.suc), dirichlet_term(f, g, n))) =
                sum(map(divisors_up_to(n, k.suc), dirichlet_cofactor_term(f, g, n)))
        }
    }
}

/// At any bound `k`, for positive `n`, the dirichlet-term sum equals the
/// cofactor-indexed swapped-pair sum.
theorem dirichlet_swap_below(f: Nat -> Nat, g: Nat -> Nat, n: Nat, k: Nat) {
    dirichlet_swap_below_pred(f, g, n, k)
} by {
    define p(x: Nat) -> Bool {
        dirichlet_swap_below_pred(f, g, n, x)
    }
    dirichlet_swap_below_base(f, g, n)
    p(Nat.0)
    forall(j: Nat) {
        if p(j) {
            dirichlet_swap_below_pred(f, g, n, j)
            dirichlet_swap_below_step(f, g, n, j)
            dirichlet_swap_below_pred(f, g, n, j.suc)
            p(j.suc)
        }
    }
    p(k)
}

/// The cofactor map at fixed `n`, viewed as a unary function from divisors of
/// `n` to divisors of `n`.
define nat_divisor_quotient_fn(n: Nat) -> (Nat -> Nat) {
    function(d: Nat) { divisor_quotient(n, d) }
}

/// Application of the unary cofactor map.
theorem nat_divisor_quotient_fn_apply(n: Nat, d: Nat) {
    nat_divisor_quotient_fn(n)(d) = divisor_quotient(n, d)
}

/// The list-image of the divisors of `n` under the cofactor map.
define cofactor_image_list(n: Nat) -> List[Nat] {
    map(divisor_list(n), nat_divisor_quotient_fn(n))
}

/// The cofactor map is an involution on divisors of positive `n`.
theorem nat_divisor_quotient_fn_involution(n: Nat, d: Nat) {
    Nat.0 < n and d.divides(n) implies
        nat_divisor_quotient_fn(n)(nat_divisor_quotient_fn(n)(d)) = d
} by {
    if Nat.0 < n and d.divides(n) {
        nat_divisor_quotient_fn_apply(n, d)
        nat_divisor_quotient_fn(n)(d) = divisor_quotient(n, d)
        nat_divisor_quotient_fn_apply(n, divisor_quotient(n, d))
        nat_divisor_quotient_fn(n)(divisor_quotient(n, d)) =
            divisor_quotient(n, divisor_quotient(n, d))
        divisor_quotient_involution(n, d)
        divisor_quotient(n, divisor_quotient(n, d)) = d
    }
}

/// The cofactor map sends divisors of positive `n` to divisors of `n`.
theorem nat_divisor_quotient_fn_divides(n: Nat, d: Nat) {
    d.divides(n) implies nat_divisor_quotient_fn(n)(d).divides(n)
} by {
    if d.divides(n) {
        nat_divisor_quotient_fn_apply(n, d)
        nat_divisor_quotient_fn(n)(d) = divisor_quotient(n, d)
        divisor_quotient_divides(n, d)
    }
}

/// Swap form of Dirichlet convolution: for positive `n`,
/// `dirichlet_convolve(f, g)(n)` equals the sum of the cofactor-indexed
/// swapped-pair term `(g, f)` over the divisor list of `n`. A stepping stone
/// toward commutativity.
theorem dirichlet_convolve_swap_form(f: Nat -> Nat, g: Nat -> Nat, n: Nat) {
    Nat.0 < n implies
        dirichlet_convolve(f, g)(n) =
        sum(map(divisor_list(n), dirichlet_cofactor_term(f, g, n)))
} by {
    if Nat.0 < n {
        dirichlet_swap_below(f, g, n, n)
        dirichlet_swap_below_pred(f, g, n, n)
        dirichlet_swap_below_pred(f, g, n, n) =
            (Nat.0 < n implies
                sum(map(divisors_up_to(n, n), dirichlet_term(f, g, n))) =
                sum(map(divisors_up_to(n, n), dirichlet_cofactor_term(f, g, n))))
        sum(map(divisors_up_to(n, n), dirichlet_term(f, g, n))) =
            sum(map(divisors_up_to(n, n), dirichlet_cofactor_term(f, g, n)))
        divisor_list(n) = divisors_up_to(n, n)
        map(divisor_list(n), dirichlet_term(f, g, n)) =
            map(divisors_up_to(n, n), dirichlet_term(f, g, n))
        map(divisor_list(n), dirichlet_cofactor_term(f, g, n)) =
            map(divisors_up_to(n, n), dirichlet_cofactor_term(f, g, n))
        dirichlet_convolve_apply(f, g, n)
        dirichlet_convolve(f, g)(n) =
            sum(map(divisor_list(n), dirichlet_term(f, g, n)))
        sum(map(divisor_list(n), dirichlet_term(f, g, n))) =
            sum(map(divisor_list(n), dirichlet_cofactor_term(f, g, n)))
        dirichlet_convolve(f, g)(n) =
            sum(map(divisor_list(n), dirichlet_cofactor_term(f, g, n)))
    }
}

/// Pointwise: the dirichlet term of a pointwise sum splits as a sum of
/// dirichlet terms in the first argument.
theorem dirichlet_term_add_left_eq(f: Nat -> Nat, g: Nat -> Nat, h: Nat -> Nat,
        n: Nat) {
    dirichlet_term(arithmetic_fn_add(f, g), h, n) =
        add_fn(dirichlet_term(f, h, n), dirichlet_term(g, h, n))
} by {
    forall(d: Nat) {
        dirichlet_term_apply(arithmetic_fn_add(f, g), h, n, d)
        dirichlet_term(arithmetic_fn_add(f, g), h, n)(d) =
            arithmetic_fn_add(f, g)(d) * h(divisor_quotient(n, d))
        arithmetic_fn_add_apply(f, g, d)
        arithmetic_fn_add(f, g)(d) = f(d) + g(d)
        dirichlet_term(arithmetic_fn_add(f, g), h, n)(d) =
            (f(d) + g(d)) * h(divisor_quotient(n, d))
        (f(d) + g(d)) * h(divisor_quotient(n, d)) =
            f(d) * h(divisor_quotient(n, d)) + g(d) * h(divisor_quotient(n, d))
        dirichlet_term_apply(f, h, n, d)
        dirichlet_term_apply(g, h, n, d)
        dirichlet_term(f, h, n)(d) = f(d) * h(divisor_quotient(n, d))
        dirichlet_term(g, h, n)(d) = g(d) * h(divisor_quotient(n, d))
        add_fn(dirichlet_term(f, h, n), dirichlet_term(g, h, n), d) =
            dirichlet_term(f, h, n)(d) + dirichlet_term(g, h, n)(d)
        dirichlet_term(arithmetic_fn_add(f, g), h, n)(d) =
            add_fn(dirichlet_term(f, h, n), dirichlet_term(g, h, n), d)
    }
}

/// Pointwise: the dirichlet term of a pointwise sum splits as a sum of
/// dirichlet terms in the second argument.
theorem dirichlet_term_add_right_eq(f: Nat -> Nat, g: Nat -> Nat, h: Nat -> Nat,
        n: Nat) {
    dirichlet_term(f, arithmetic_fn_add(g, h), n) =
        add_fn(dirichlet_term(f, g, n), dirichlet_term(f, h, n))
} by {
    forall(d: Nat) {
        dirichlet_term_apply(f, arithmetic_fn_add(g, h), n, d)
        dirichlet_term(f, arithmetic_fn_add(g, h), n)(d) =
            f(d) * arithmetic_fn_add(g, h)(divisor_quotient(n, d))
        arithmetic_fn_add_apply(g, h, divisor_quotient(n, d))
        arithmetic_fn_add(g, h)(divisor_quotient(n, d)) =
            g(divisor_quotient(n, d)) + h(divisor_quotient(n, d))
        dirichlet_term(f, arithmetic_fn_add(g, h), n)(d) =
            f(d) * (g(divisor_quotient(n, d)) + h(divisor_quotient(n, d)))
        f(d) * (g(divisor_quotient(n, d)) + h(divisor_quotient(n, d))) =
            f(d) * g(divisor_quotient(n, d)) + f(d) * h(divisor_quotient(n, d))
        dirichlet_term_apply(f, g, n, d)
        dirichlet_term_apply(f, h, n, d)
        dirichlet_term(f, g, n)(d) = f(d) * g(divisor_quotient(n, d))
        dirichlet_term(f, h, n)(d) = f(d) * h(divisor_quotient(n, d))
        add_fn(dirichlet_term(f, g, n), dirichlet_term(f, h, n), d) =
            dirichlet_term(f, g, n)(d) + dirichlet_term(f, h, n)(d)
        dirichlet_term(f, arithmetic_fn_add(g, h), n)(d) =
            add_fn(dirichlet_term(f, g, n), dirichlet_term(f, h, n), d)
    }
}

/// Dirichlet convolution distributes over pointwise addition on the left:
/// `(f + g) * h = f * h + g * h`.
theorem dirichlet_convolve_add_left(f: Nat -> Nat, g: Nat -> Nat, h: Nat -> Nat) {
    dirichlet_convolve(arithmetic_fn_add(f, g), h) =
        arithmetic_fn_add(dirichlet_convolve(f, h), dirichlet_convolve(g, h))
} by {
    forall(n: Nat) {
        dirichlet_convolve_apply(arithmetic_fn_add(f, g), h, n)
        dirichlet_convolve(arithmetic_fn_add(f, g), h)(n) =
            sum(map(divisor_list(n), dirichlet_term(arithmetic_fn_add(f, g), h, n)))
        dirichlet_term_add_left_eq(f, g, h, n)
        dirichlet_term(arithmetic_fn_add(f, g), h, n) =
            add_fn(dirichlet_term(f, h, n), dirichlet_term(g, h, n))
        map(divisor_list(n), dirichlet_term(arithmetic_fn_add(f, g), h, n)) =
            map(divisor_list(n),
                add_fn(dirichlet_term(f, h, n), dirichlet_term(g, h, n)))
        map_sum_add(divisor_list(n), dirichlet_term(f, h, n),
            dirichlet_term(g, h, n))
        sum(map(divisor_list(n), dirichlet_term(f, h, n))) +
            sum(map(divisor_list(n), dirichlet_term(g, h, n))) =
            sum(map(divisor_list(n),
                add_fn(dirichlet_term(f, h, n), dirichlet_term(g, h, n))))
        dirichlet_convolve_apply(f, h, n)
        dirichlet_convolve_apply(g, h, n)
        dirichlet_convolve(f, h)(n) =
            sum(map(divisor_list(n), dirichlet_term(f, h, n)))
        dirichlet_convolve(g, h)(n) =
            sum(map(divisor_list(n), dirichlet_term(g, h, n)))
        dirichlet_convolve(arithmetic_fn_add(f, g), h)(n) =
            dirichlet_convolve(f, h)(n) + dirichlet_convolve(g, h)(n)
        arithmetic_fn_add_apply(dirichlet_convolve(f, h),
            dirichlet_convolve(g, h), n)
        arithmetic_fn_add(dirichlet_convolve(f, h), dirichlet_convolve(g, h))(n) =
            dirichlet_convolve(f, h)(n) + dirichlet_convolve(g, h)(n)
        dirichlet_convolve(arithmetic_fn_add(f, g), h)(n) =
            arithmetic_fn_add(dirichlet_convolve(f, h), dirichlet_convolve(g, h))(n)
    }
}

/// Dirichlet convolution distributes over pointwise addition on the right:
/// `f * (g + h) = f * g + f * h`.
theorem dirichlet_convolve_add_right(f: Nat -> Nat, g: Nat -> Nat, h: Nat -> Nat) {
    dirichlet_convolve(f, arithmetic_fn_add(g, h)) =
        arithmetic_fn_add(dirichlet_convolve(f, g), dirichlet_convolve(f, h))
} by {
    forall(n: Nat) {
        dirichlet_convolve_apply(f, arithmetic_fn_add(g, h), n)
        dirichlet_convolve(f, arithmetic_fn_add(g, h))(n) =
            sum(map(divisor_list(n), dirichlet_term(f, arithmetic_fn_add(g, h), n)))
        dirichlet_term_add_right_eq(f, g, h, n)
        dirichlet_term(f, arithmetic_fn_add(g, h), n) =
            add_fn(dirichlet_term(f, g, n), dirichlet_term(f, h, n))
        map(divisor_list(n), dirichlet_term(f, arithmetic_fn_add(g, h), n)) =
            map(divisor_list(n),
                add_fn(dirichlet_term(f, g, n), dirichlet_term(f, h, n)))
        map_sum_add(divisor_list(n), dirichlet_term(f, g, n),
            dirichlet_term(f, h, n))
        sum(map(divisor_list(n), dirichlet_term(f, g, n))) +
            sum(map(divisor_list(n), dirichlet_term(f, h, n))) =
            sum(map(divisor_list(n),
                add_fn(dirichlet_term(f, g, n), dirichlet_term(f, h, n))))
        dirichlet_convolve_apply(f, g, n)
        dirichlet_convolve_apply(f, h, n)
        dirichlet_convolve(f, g)(n) =
            sum(map(divisor_list(n), dirichlet_term(f, g, n)))
        dirichlet_convolve(f, h)(n) =
            sum(map(divisor_list(n), dirichlet_term(f, h, n)))
        dirichlet_convolve(f, arithmetic_fn_add(g, h))(n) =
            dirichlet_convolve(f, g)(n) + dirichlet_convolve(f, h)(n)
        arithmetic_fn_add_apply(dirichlet_convolve(f, g),
            dirichlet_convolve(f, h), n)
        arithmetic_fn_add(dirichlet_convolve(f, g), dirichlet_convolve(f, h))(n) =
            dirichlet_convolve(f, g)(n) + dirichlet_convolve(f, h)(n)
        dirichlet_convolve(f, arithmetic_fn_add(g, h))(n) =
            arithmetic_fn_add(dirichlet_convolve(f, g), dirichlet_convolve(f, h))(n)
    }
}

/// For positive `n`, the cofactor image of the divisor list and the divisor
/// list itself contain exactly the same elements: the positive divisors of `n`.
theorem cofactor_image_list_contains_iff(n: Nat, x: Nat) {
    Nat.0 < n implies
        (cofactor_image_list(n).contains(x) = divisor_list(n).contains(x))
} by {
    if Nat.0 < n {
        cofactor_image_list(n) = map(divisor_list(n), nat_divisor_quotient_fn(n))
        if cofactor_image_list(n).contains(x) {
            map(divisor_list(n), nat_divisor_quotient_fn(n)).contains(x)
            map_contains(divisor_list(n), nat_divisor_quotient_fn(n), x)
            let d: Nat satisfy {
                divisor_list(n).contains(d) and nat_divisor_quotient_fn(n)(d) = x
            }
            divisor_list_contains_implies(n, d)
            Nat.0 < d
            d.divides(n)
            nat_divisor_quotient_fn_apply(n, d)
            nat_divisor_quotient_fn(n)(d) = divisor_quotient(n, d)
            divisor_quotient(n, d) = x
            divisor_quotient_divides(n, d)
            divisor_quotient(n, d).divides(n)
            x.divides(n)
            divisor_quotient_positive(n, d)
            Nat.0 < divisor_quotient(n, d)
            Nat.0 < x
            divisor_list_contains_of(n, x)
            divisor_list(n).contains(x)
        }
        if divisor_list(n).contains(x) {
            divisor_list_contains_implies(n, x)
            Nat.0 < x
            x.divides(n)
            divisor_quotient_divides(n, x)
            divisor_quotient(n, x).divides(n)
            divisor_quotient_positive(n, x)
            Nat.0 < divisor_quotient(n, x)
            divisor_list_contains_of(n, divisor_quotient(n, x))
            divisor_list(n).contains(divisor_quotient(n, x))
            map_contains_of_contains(divisor_list(n), nat_divisor_quotient_fn(n),
                divisor_quotient(n, x))
            map(divisor_list(n), nat_divisor_quotient_fn(n)).contains(
                nat_divisor_quotient_fn(n)(divisor_quotient(n, x)))
            nat_divisor_quotient_fn_apply(n, divisor_quotient(n, x))
            nat_divisor_quotient_fn(n)(divisor_quotient(n, x)) =
                divisor_quotient(n, divisor_quotient(n, x))
            divisor_quotient_involution(n, x)
            divisor_quotient(n, divisor_quotient(n, x)) = x
            nat_divisor_quotient_fn(n)(divisor_quotient(n, x)) = x
            map(divisor_list(n), nat_divisor_quotient_fn(n)).contains(x)
            cofactor_image_list(n).contains(x)
        }
        cofactor_image_list(n).contains(x) implies divisor_list(n).contains(x)
        divisor_list(n).contains(x) implies cofactor_image_list(n).contains(x)
        cofactor_image_list(n).contains(x) = divisor_list(n).contains(x)
    }
}

/// Cons-step uniqueness lemma: a unique tail with a fresh head produces a
/// unique list.
theorem cons_unique_of_fresh(head: Nat, tail: List[Nat]) {
    tail.is_unique and not tail.contains(head)
        implies List.cons(head, tail).is_unique
} by {
    if tail.is_unique and not tail.contains(head) {
        tail.unique = tail
        List.cons(head, tail).unique = List.cons(head, tail.unique)
        List.cons(head, tail).unique = List.cons(head, tail)
        List.cons(head, tail).is_unique
    }
}

/// The cofactor image of a sublist of the divisor list whose entries inject
/// (under the cofactor map) is itself unique.
define cofactor_image_unique_pred(n: Nat) -> (List[Nat] -> Bool) {
    function(l: List[Nat]) {
        l.is_unique and (forall(d: Nat) { l.contains(d) implies d.divides(n) })
            and Nat.0 < n
            implies map(l, nat_divisor_quotient_fn(n)).is_unique
    }
}

/// Base case: the empty list maps to the empty list.
theorem cofactor_image_unique_base(n: Nat) {
    cofactor_image_unique_pred(n)(List.nil[Nat])
} by {
    map[Nat, Nat](List.nil[Nat], nat_divisor_quotient_fn(n)) = List.nil[Nat]
    List.nil[Nat].unique = List.nil[Nat]
    List.nil[Nat].is_unique
}

/// Step case: prepending a divisor `head` not already in `tail` to a unique
/// sublist of divisors yields a unique list whose cofactor image is unique.
theorem cofactor_image_unique_step(n: Nat, head: Nat, tail: List[Nat]) {
    cofactor_image_unique_pred(n)(tail)
        implies cofactor_image_unique_pred(n)(List.cons(head, tail))
} by {
    if cofactor_image_unique_pred(n)(tail) {
        cofactor_image_unique_pred(n)(tail) =
            (tail.is_unique and
                (forall(d: Nat) { tail.contains(d) implies d.divides(n) })
                and Nat.0 < n
                implies map(tail, nat_divisor_quotient_fn(n)).is_unique)
        if List.cons(head, tail).is_unique and
                (forall(d: Nat) { List.cons(head, tail).contains(d) implies d.divides(n) })
                and Nat.0 < n {
            // tail is unique
            tail.is_unique
            forall(d: Nat) {
                if tail.contains(d) {
                    List.cons(head, tail).contains(d)
                    d.divides(n)
                }
            }
            map(tail, nat_divisor_quotient_fn(n)).is_unique
            // head is a divisor
            List.cons(head, tail).contains(head)
            head.divides(n)
            // head not in tail (since cons is unique)
            List.cons(head, tail).unique = List.cons(head, tail)
            if tail.contains(head) {
                List.cons(head, tail).unique = tail.unique
                tail.unique = List.cons(head, tail)
                tail.unique.length = List.cons(head, tail).length
                List.cons(head, tail).length = tail.length.suc
                tail.unique.length = tail.length.suc
                tail.unique.length <= tail.length
                tail.length.suc <= tail.length
                false
            }
            not tail.contains(head)
            // Now show f(head) not in map(tail, f)
            map[Nat, Nat](List.cons(head, tail), nat_divisor_quotient_fn(n)) =
                List.cons(nat_divisor_quotient_fn(n)(head),
                    map(tail, nat_divisor_quotient_fn(n)))
            nat_divisor_quotient_fn_apply(n, head)
            nat_divisor_quotient_fn(n)(head) = divisor_quotient(n, head)
            if map(tail, nat_divisor_quotient_fn(n)).contains(divisor_quotient(n, head)) {
                map_contains(tail, nat_divisor_quotient_fn(n), divisor_quotient(n, head))
                let d2: Nat satisfy {
                    tail.contains(d2) and nat_divisor_quotient_fn(n)(d2) = divisor_quotient(n, head)
                }
                nat_divisor_quotient_fn_apply(n, d2)
                nat_divisor_quotient_fn(n)(d2) = divisor_quotient(n, d2)
                divisor_quotient(n, d2) = divisor_quotient(n, head)
                tail.contains(d2)
                List.cons(head, tail).contains(d2)
                d2.divides(n)
                // apply involution: divisor_quotient(n, divisor_quotient(n, d2)) = d2
                divisor_quotient_involution(n, d2)
                divisor_quotient(n, divisor_quotient(n, d2)) = d2
                divisor_quotient_involution(n, head)
                divisor_quotient(n, divisor_quotient(n, head)) = head
                divisor_quotient(n, divisor_quotient(n, d2)) =
                    divisor_quotient(n, divisor_quotient(n, head))
                d2 = head
                tail.contains(head)
                false
            }
            not map(tail, nat_divisor_quotient_fn(n)).contains(divisor_quotient(n, head))
            not map(tail, nat_divisor_quotient_fn(n)).contains(nat_divisor_quotient_fn(n)(head))
            cons_unique_of_fresh(nat_divisor_quotient_fn(n)(head),
                map(tail, nat_divisor_quotient_fn(n)))
            List.cons(nat_divisor_quotient_fn(n)(head),
                map(tail, nat_divisor_quotient_fn(n))).is_unique
            map(List.cons(head, tail), nat_divisor_quotient_fn(n)).is_unique
        }
    }
}

/// The cofactor image of any unique sublist of divisors of positive `n` is
/// unique.
theorem cofactor_image_unique_thm(n: Nat, l: List[Nat]) {
    cofactor_image_unique_pred(n)(l)
} by {
    cofactor_image_unique_base(n)
    forall(head: Nat, tail: List[Nat]) {
        if cofactor_image_unique_pred(n)(tail) {
            cofactor_image_unique_step(n, head, tail)
        }
    }
    List.induction(cofactor_image_unique_pred(n))
}

/// For positive `n`, the cofactor image of the divisor list is itself a unique
/// list.
theorem cofactor_image_list_is_unique(n: Nat) {
    Nat.0 < n implies cofactor_image_list(n).is_unique
} by {
    if Nat.0 < n {
        cofactor_image_unique_thm(n, divisor_list(n))
        cofactor_image_unique_pred(n)(divisor_list(n)) =
            (divisor_list(n).is_unique and
                (forall(d: Nat) { divisor_list(n).contains(d) implies d.divides(n) })
                and Nat.0 < n
                implies map(divisor_list(n), nat_divisor_quotient_fn(n)).is_unique)
        divisor_list_is_unique(n)
        forall(d: Nat) {
            if divisor_list(n).contains(d) {
                divisor_list_contains_implies(n, d)
                d.divides(n)
            }
        }
        map(divisor_list(n), nat_divisor_quotient_fn(n)).is_unique
        cofactor_image_list(n).is_unique
    }
}

/// The cofactor map composed with `dirichlet_term(g, f, n)` over the divisor
/// list. Inductive predicate driving the composition rewrite.
define cofactor_term_compose_pred(f: Nat -> Nat, g: Nat -> Nat, n: Nat) -> (List[Nat] -> Bool) {
    function(l: List[Nat]) {
        map(l, dirichlet_cofactor_term(f, g, n)) =
            map(map(l, nat_divisor_quotient_fn(n)), dirichlet_term(g, f, n))
    }
}

/// Base case for the cofactor-composition rewrite.
theorem cofactor_term_compose_base(f: Nat -> Nat, g: Nat -> Nat, n: Nat) {
    cofactor_term_compose_pred(f, g, n)(List.nil[Nat])
} by {
    map[Nat, Nat](List.nil[Nat], dirichlet_cofactor_term(f, g, n)) = List.nil[Nat]
    map[Nat, Nat](List.nil[Nat], nat_divisor_quotient_fn(n)) = List.nil[Nat]
    map[Nat, Nat](List.nil[Nat], dirichlet_term(g, f, n)) = List.nil[Nat]
}

/// Step case for the cofactor-composition rewrite.
theorem cofactor_term_compose_step(f: Nat -> Nat, g: Nat -> Nat, n: Nat,
        head: Nat, tail: List[Nat]) {
    cofactor_term_compose_pred(f, g, n)(tail)
        implies cofactor_term_compose_pred(f, g, n)(List.cons(head, tail))
} by {
    if cofactor_term_compose_pred(f, g, n)(tail) {
        map(tail, dirichlet_cofactor_term(f, g, n)) =
            map(map(tail, nat_divisor_quotient_fn(n)), dirichlet_term(g, f, n))
        map[Nat, Nat](List.cons(head, tail), dirichlet_cofactor_term(f, g, n)) =
            List.cons(dirichlet_cofactor_term(f, g, n)(head),
                map(tail, dirichlet_cofactor_term(f, g, n)))
        map[Nat, Nat](List.cons(head, tail), nat_divisor_quotient_fn(n)) =
            List.cons(nat_divisor_quotient_fn(n)(head),
                map(tail, nat_divisor_quotient_fn(n)))
        map[Nat, Nat](List.cons(nat_divisor_quotient_fn(n)(head),
                map(tail, nat_divisor_quotient_fn(n))), dirichlet_term(g, f, n)) =
            List.cons(dirichlet_term(g, f, n)(nat_divisor_quotient_fn(n)(head)),
                map(map(tail, nat_divisor_quotient_fn(n)), dirichlet_term(g, f, n)))
        dirichlet_cofactor_term_apply(f, g, n, head)
        nat_divisor_quotient_fn_apply(n, head)
        dirichlet_cofactor_term(f, g, n)(head) =
            dirichlet_term(g, f, n)(divisor_quotient(n, head))
        dirichlet_term(g, f, n)(nat_divisor_quotient_fn(n)(head)) =
            dirichlet_term(g, f, n)(divisor_quotient(n, head))
        dirichlet_cofactor_term(f, g, n)(head) =
            dirichlet_term(g, f, n)(nat_divisor_quotient_fn(n)(head))
        map(List.cons(head, tail), dirichlet_cofactor_term(f, g, n)) =
            List.cons(dirichlet_term(g, f, n)(nat_divisor_quotient_fn(n)(head)),
                map(map(tail, nat_divisor_quotient_fn(n)), dirichlet_term(g, f, n)))
        map(List.cons(head, tail), dirichlet_cofactor_term(f, g, n)) =
            map(List.cons(nat_divisor_quotient_fn(n)(head),
                map(tail, nat_divisor_quotient_fn(n))), dirichlet_term(g, f, n))
        map(List.cons(head, tail), dirichlet_cofactor_term(f, g, n)) =
            map(map(List.cons(head, tail), nat_divisor_quotient_fn(n)),
                dirichlet_term(g, f, n))
    }
}

/// The cofactor-indexed term sum equals the dirichlet-term sum over the
/// cofactor image.
theorem cofactor_term_compose(f: Nat -> Nat, g: Nat -> Nat, n: Nat, l: List[Nat]) {
    map(l, dirichlet_cofactor_term(f, g, n)) =
        map(map(l, nat_divisor_quotient_fn(n)), dirichlet_term(g, f, n))
} by {
    cofactor_term_compose_base(f, g, n)
    forall(head: Nat, tail: List[Nat]) {
        if cofactor_term_compose_pred(f, g, n)(tail) {
            cofactor_term_compose_step(f, g, n, head, tail)
        }
    }
    List.induction(cofactor_term_compose_pred(f, g, n))
    cofactor_term_compose_pred(f, g, n)(l)
}

/// Dirichlet convolution is commutative on positive arguments.
theorem dirichlet_convolve_comm_positive(f: Nat -> Nat, g: Nat -> Nat, n: Nat) {
    Nat.0 < n implies dirichlet_convolve(f, g)(n) = dirichlet_convolve(g, f)(n)
} by {
    if Nat.0 < n {
        dirichlet_convolve_swap_form(f, g, n)
        dirichlet_convolve(f, g)(n) =
            sum(map(divisor_list(n), dirichlet_cofactor_term(f, g, n)))
        cofactor_term_compose(f, g, n, divisor_list(n))
        map(divisor_list(n), dirichlet_cofactor_term(f, g, n)) =
            map(map(divisor_list(n), nat_divisor_quotient_fn(n)),
                dirichlet_term(g, f, n))
        map(divisor_list(n), dirichlet_cofactor_term(f, g, n)) =
            map(cofactor_image_list(n), dirichlet_term(g, f, n))
        sum(map(divisor_list(n), dirichlet_cofactor_term(f, g, n))) =
            sum(map(cofactor_image_list(n), dirichlet_term(g, f, n)))
        // now use unique_same_contains_map_sum_eq with cofactor_image_list and divisor_list
        divisor_list_is_unique(n)
        cofactor_image_list_is_unique(n)
        forall(x: Nat) {
            cofactor_image_list_contains_iff(n, x)
            cofactor_image_list(n).contains(x) = divisor_list(n).contains(x)
        }
        unique_same_contains_map_sum_eq(cofactor_image_list(n), divisor_list(n),
            dirichlet_term(g, f, n))
        sum(map(cofactor_image_list(n), dirichlet_term(g, f, n))) =
            sum(map(divisor_list(n), dirichlet_term(g, f, n)))
        dirichlet_convolve_apply(g, f, n)
        dirichlet_convolve(g, f)(n) =
            sum(map(divisor_list(n), dirichlet_term(g, f, n)))
        dirichlet_convolve(f, g)(n) = dirichlet_convolve(g, f)(n)
    }
}

/// Dirichlet convolution is commutative.
theorem dirichlet_convolve_comm(f: Nat -> Nat, g: Nat -> Nat) {
    dirichlet_convolve(f, g) = dirichlet_convolve(g, f)
} by {
    forall(n: Nat) {
        if n = Nat.0 {
            dirichlet_convolve_at_zero(f, g)
            dirichlet_convolve_at_zero(g, f)
            dirichlet_convolve(f, g)(n) = Nat.0
            dirichlet_convolve(g, f)(n) = Nat.0
            dirichlet_convolve(f, g)(n) = dirichlet_convolve(g, f)(n)
        } else {
            Nat.0 < n
            dirichlet_convolve_comm_positive(f, g, n)
            dirichlet_convolve(f, g)(n) = dirichlet_convolve(g, f)(n)
        }
    }
}
