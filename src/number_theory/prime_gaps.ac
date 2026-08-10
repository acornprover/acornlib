from nat import Nat, lt_or_lte, lt_suc_right, lt_imp_lte_suc,
    lt_trans, lte_and_lt, lt_and_lte, lt_add_left, lt_suc, lte_add_right,
    lt_diff, lt_not_ref, lte_suc_suc,
    add_comm, add_assoc, add_one_right, add_cancels_left, add_cancels_right,
    distrib_left, lte_one_factorial, divides_factorial, pos_of_ne_zero, not_lt_zero,
    lte_imp_not_lt, lte_ref, two_divides_suc_iff, has_min,
    is_min, is_min_apply, is_min_false_below, false_below, false_below_apply,
    exists_infinite_primes, mul_one_right, add_zero_right
from combinatorics import factorial_nonzero
from number_theory import nat_two_prime
from data.nat.nat_bounded_max import is_max, has_max, is_upper_bound_of,
    is_upper_bound_of_intro, is_max_apply, is_max_is_upper_bound

numerals Nat

/// A prime divisible by two is two.
theorem even_prime_eq_two(p: Nat) {
    p.is_prime and Nat.2.divides(p) implies p = Nat.2
} by {
    if p.is_prime and Nat.2.divides(p) {
        Nat.2.divides(p) = exists(c: Nat) { Nat.2 * c = p }
        let c: Nat satisfy { Nat.2 * c = p }
        if c = Nat.0 {
            Nat.2 * c = Nat.0
            p = Nat.0
            (p.is_prime = (Nat.1 < p and not p.is_composite))
            Nat.1 < p
            Nat.1 < Nat.0
            not_lt_zero(Nat.1)
            not (Nat.1 < Nat.0)
            false
        }
        c != Nat.0
        if Nat.1 < c {
            exists(b: Nat, c2: Nat) { Nat.1 < b and Nat.1 < c2 and p = b * c2 }
            p.is_composite
            p.is_prime = Nat.1 < p and not p.is_composite
            not p.is_composite
            false
        }
        lt_or_lte(Nat.1, c)
        if Nat.1 < c {
            false
        }
        c <= Nat.1
        lt_suc_right(c, Nat.1)
        if c < Nat.1 {
            lt_suc_right(c, Nat.0)
            if c = Nat.0 {
                false
            }
            not_lt_zero(c)
            false
        }
        c = Nat.1
        Nat.2 * c = Nat.2
        p = Nat.2
    }
}

/// The classical construction: every number `(n+1)! + k` with `2 <= k <= n+1` is
/// composite.
///
/// Since `k <= n+1`, the factorial `(n+1)!` contains `k` as a factor, so `k` divides
/// `(n+1)! + k`; and the complementary factor `(n+1)!/k + 1` is greater than one.
theorem factorial_shift_composite(n: Nat, k: Nat) {
    Nat.2 <= k and k <= n.suc implies (n.suc.factorial + k).is_composite
} by {
    if Nat.2 <= k and k <= n.suc {
        divides_factorial(k, n.suc)
        (k != Nat.0 and k <= n.suc implies k.divides(n.suc.factorial))
        lt_suc(Nat.0)
        Nat.0 < Nat.1
        lt_suc(Nat.1)
        Nat.1 < Nat.2
        lt_trans(Nat.0, Nat.1, Nat.2)
        Nat.0 < Nat.2
        if k = Nat.0 {
            Nat.2 <= Nat.0
            lte_imp_not_lt(Nat.2, Nat.0)
            not (Nat.0 < Nat.2)
            false
        }
        k != Nat.0
        lt_and_lte(Nat.1, Nat.2, k)
        Nat.1 < k
        k.divides(n.suc.factorial)
        (k.divides(n.suc.factorial) = exists(c: Nat) { k * c = n.suc.factorial })
        let (q: Nat) satisfy { k * q = n.suc.factorial }
        Nat.1 < k
        factorial_nonzero(n.suc)
        n.suc.factorial != Nat.0
        if q = Nat.0 {
            k * q = k * Nat.0
            k * Nat.0 = Nat.0
            n.suc.factorial = Nat.0
            false
        }
        q != Nat.0
        pos_of_ne_zero(q)
        Nat.0 < q
        lt_add_left(Nat.1, Nat.0, q)
        Nat.1 + Nat.0 < Nat.1 + q
        add_zero_right(Nat.1)
        Nat.1 < Nat.1 + q
        add_comm(Nat.1, q)
        Nat.1 < q + Nat.1
        distrib_left(k, q, Nat.1)
        k * (q + Nat.1) = k * q + k * Nat.1
        mul_one_right(k)
        k * Nat.1 = k
        k * (q + Nat.1) = k * q + k
        k * q = n.suc.factorial
        k * (q + Nat.1) = n.suc.factorial + k
        exists(b: Nat, c: Nat) {
            Nat.1 < b and Nat.1 < c and n.suc.factorial + k = b * c
        }
        (n.suc.factorial + k).is_composite = exists(b: Nat, c: Nat) {
            Nat.1 < b and Nat.1 < c and n.suc.factorial + k = b * c
        }
        (n.suc.factorial + k).is_composite
    }
}

/// The only two consecutive primes are 2 and 3.
///
/// Of two consecutive naturals one is even, and an even prime must be two, so the
/// smaller prime is 2 (and the larger, 3).
theorem consecutive_primes_differ_one(p: Nat) {
    p.is_prime and p.suc.is_prime implies p = Nat.2
} by {
    if p.is_prime and p.suc.is_prime {
        two_divides_suc_iff(p)
        Nat.2.divides(p.suc) = not Nat.2.divides(p)
        if Nat.2.divides(p) {
            even_prime_eq_two(p)
            (p.is_prime and Nat.2.divides(p) implies p = Nat.2)
            p = Nat.2
        }
        if not Nat.2.divides(p) {
            Nat.2.divides(p.suc)
            even_prime_eq_two(p.suc)
            (p.suc.is_prime and Nat.2.divides(p.suc) implies p.suc = Nat.2)
            p.suc = Nat.2
            add_one_right(p)
            (p + Nat.1 = p.suc)
            add_one_right(Nat.1)
            (Nat.1 + Nat.1 = Nat.1.suc)
            p + Nat.1 = Nat.2
            Nat.1.suc = Nat.2
            Nat.1 + Nat.1 = Nat.2
            add_cancels_right(Nat.1, p, Nat.1)
            p = Nat.1
            (p.is_prime = (Nat.1 < p and not p.is_composite))
            Nat.1 < p
            Nat.1 < Nat.1
            lte_ref(Nat.1)
            Nat.1 <= Nat.1
            lte_imp_not_lt(Nat.1, Nat.1)
            not (Nat.1 < Nat.1)
            false
        }
        p = Nat.2
    }
}

/// If two primes differ by one, they are 2 and 3.
theorem only_prime_pair_differ_one(p: Nat, q: Nat) {
    p.is_prime and q.is_prime and q = p + Nat.1 implies p = Nat.2 and q = Nat.3
} by {
    if p.is_prime and q.is_prime and q = p + Nat.1 {
        add_one_right(p)
        (p + Nat.1 = p.suc)
        q = p.suc
        p.suc.is_prime
        consecutive_primes_differ_one(p)
        (p.is_prime and p.suc.is_prime implies p = Nat.2)
        p = Nat.2
        p.suc = Nat.3
        q = Nat.3
        p = Nat.2 and q = Nat.3
    }
}

/// Euclid's theorem: there are infinitely many primes.
///
/// Beyond every natural number there is a larger prime, proved by considering a
/// prime factor of `n! + 1`.
theorem infinitely_many_primes(n: Nat) {
    exists(p: Nat) {
        n < p and p.is_prime
    }
} by {
    exists_infinite_primes(n)
}

/// For every `n` there is a run of `n` consecutive composite numbers.
///
/// The classical construction: `(n+1)! + 2`, `(n+1)! + 3`, ..., `(n+1)! + (n+1)`
/// are all composite.
theorem consecutive_composite_run(n: Nat) {
    exists(x: Nat) {
        forall(k: Nat) {
            Nat.2 <= k and k <= n.suc implies (x + k).is_composite
        }
    }
} by {
    forall(k: Nat) {
        if Nat.2 <= k and k <= n.suc {
            factorial_shift_composite(n, k)
            (Nat.2 <= k and k <= n.suc implies (n.suc.factorial + k).is_composite)
            (n.suc.factorial + k).is_composite
        }
        (Nat.2 <= k and k <= n.suc implies (n.suc.factorial + k).is_composite)
    }
    exists(x: Nat) {
        forall(k: Nat) {
            Nat.2 <= k and k <= n.suc implies (x + k).is_composite
        }
    }
}

// ---------------------------------------------------------------------------
// Prime gaps: the classical factorial construction makes the gap between
// consecutive primes arbitrarily large.
// ---------------------------------------------------------------------------

/// True if `m` is a prime at most `y`.
define prime_at_most(y: Nat, m: Nat) -> Bool {
    m.is_prime and m <= y
}

/// The condition, spelled out.
theorem prime_at_most_eq(y: Nat, m: Nat) {
    prime_at_most(y)(m) = (m.is_prime and m <= y)
}

/// True if `q` is a prime strictly above `p`.
define prime_above(p: Nat, q: Nat) -> Bool {
    q.is_prime and p < q
}

/// The condition, spelled out.
theorem prime_above_eq(p: Nat, q: Nat) {
    prime_above(p)(q) = (q.is_prime and p < q)
}

/// A satisfiable predicate on the naturals bounded above by `y` has a greatest
/// satisfying natural; applied to the primes at most `y`.
theorem largest_prime_at_most_exists(y: Nat) {
    Nat.2 <= y implies exists(m: Nat) { is_max(prime_at_most(y), m) }
} by {
    if Nat.2 <= y {
        nat_two_prime
        Nat.2.is_prime
        prime_at_most_eq(y, Nat.2)
        (prime_at_most(y)(Nat.2) = (Nat.2.is_prime and Nat.2 <= y))
        prime_at_most(y)(Nat.2)
        forall(m: Nat) {
            if prime_at_most(y)(m) {
                prime_at_most_eq(y, m)
                (prime_at_most(y)(m) = (m.is_prime and m <= y))
                m <= y
            }
            (prime_at_most(y)(m) implies m <= y)
        }
        is_upper_bound_of_intro(prime_at_most(y), y)
        is_upper_bound_of(prime_at_most(y), y)
        has_max(prime_at_most(y), Nat.2, y)
        (prime_at_most(y)(Nat.2) and is_upper_bound_of(prime_at_most(y), y)
            implies exists(m: Nat) { is_max(prime_at_most(y), m) })
        exists(m: Nat) { is_max(prime_at_most(y), m) }
    }
}

/// The primes above `p` have a least element.
theorem least_prime_above_exists(p: Nat) {
    exists(q: Nat) { is_min(prime_above(p), q) }
} by {
    exists_infinite_primes(p)
    exists(r: Nat) { p < r and r.is_prime }
    let (r: Nat) satisfy { p < r and r.is_prime }
    prime_above_eq(p, r)
    (prime_above(p)(r) = (r.is_prime and p < r))
    prime_above(p)(r)
    has_min(prime_above(p), r)
    (prime_above(p)(r) implies exists(m: Nat) { is_min(prime_above(p), m) })
    exists(m: Nat) { is_min(prime_above(p), m) }
}

/// A prime above the greatest prime at most `(n+1)! + 1` lies above `(n+1)! + 1`.
theorem max_prime_below_above(n: Nat, p: Nat, m: Nat) {
    is_max(prime_at_most(n.suc.factorial + Nat.1), p) and m.is_prime and p < m
    implies n.suc.factorial + Nat.1 < m
} by {
    if is_max(prime_at_most(n.suc.factorial + Nat.1), p) and m.is_prime and p < m {
        if m <= n.suc.factorial + Nat.1 {
            prime_at_most_eq(n.suc.factorial + Nat.1, m)
            (prime_at_most(n.suc.factorial + Nat.1)(m) =
                (m.is_prime and m <= n.suc.factorial + Nat.1))
            prime_at_most(n.suc.factorial + Nat.1)(m)
            is_max_is_upper_bound(prime_at_most(n.suc.factorial + Nat.1), p, m)
            m <= p
            lt_and_lte(p, m, p)
            p < p
            lt_not_ref(p)
            not (p < p)
            false
        }
        not (m <= n.suc.factorial + Nat.1)
        lt_or_lte(n.suc.factorial + Nat.1, m)
        (n.suc.factorial + Nat.1 < m or m <= n.suc.factorial + Nat.1)
        n.suc.factorial + Nat.1 < m
    }
}

/// Every number in the interval `((n+1)! + 1, (n+1)! + n + 1]` is composite.
///
/// Such a number is `(n+1)! + k` for some `k` with `2 <= k <= n+1`, so the
/// classical construction applies.
theorem factorial_shift_interval_composite(n: Nat, m: Nat) {
    n.suc.factorial + Nat.1 < m and m <= n.suc.factorial + n.suc
    implies m.is_composite
} by {
    if n.suc.factorial + Nat.1 < m and m <= n.suc.factorial + n.suc {
        lt_diff(n.suc.factorial + Nat.1, m)
        (n.suc.factorial + Nat.1 < m implies
            exists(c: Nat) { n.suc.factorial + Nat.1 + c = m and c != Nat.0 })
        let (c: Nat) satisfy {
            n.suc.factorial + Nat.1 + c = m and c != Nat.0
        }
        m = n.suc.factorial + Nat.1 + c
        pos_of_ne_zero(c)
        Nat.0 < c
        lt_add_left(Nat.1, Nat.0, c)
        Nat.1 + Nat.0 < Nat.1 + c
        add_zero_right(Nat.1)
        Nat.1 < Nat.1 + c
        lt_imp_lte_suc(Nat.1, Nat.1 + c)
        Nat.2 <= Nat.1 + c
        (m <= n.suc.factorial + n.suc) = exists(d: Nat) {
            m + d = n.suc.factorial + n.suc
        }
        let (d: Nat) satisfy { m + d = n.suc.factorial + n.suc }
        add_assoc(n.suc.factorial + Nat.1, c, d)
        (n.suc.factorial + Nat.1 + c) + d =
            n.suc.factorial + Nat.1 + (c + d)
        m + d = n.suc.factorial + Nat.1 + (c + d)
        m + d = n.suc.factorial + n.suc
        n.suc.factorial + Nat.1 + (c + d) = n.suc.factorial + n.suc
        add_assoc(n.suc.factorial, Nat.1, c + d)
        n.suc.factorial + Nat.1 + (c + d) =
            n.suc.factorial + (Nat.1 + (c + d))
        n.suc.factorial + (Nat.1 + (c + d)) = n.suc.factorial + n.suc
        add_cancels_left(n.suc.factorial, Nat.1 + (c + d), n.suc)
        Nat.1 + (c + d) = n.suc
        add_assoc(Nat.1, c, d)
        (Nat.1 + c) + d = Nat.1 + (c + d)
        (Nat.1 + c) + d = n.suc
        exists(d2: Nat) { (Nat.1 + c) + d2 = n.suc }
        Nat.1 + c <= n.suc
        add_assoc(n.suc.factorial, Nat.1, c)
        (n.suc.factorial + Nat.1) + c = n.suc.factorial + (Nat.1 + c)
        m = n.suc.factorial + (Nat.1 + c)
        factorial_shift_composite(n, Nat.1 + c)
        (Nat.2 <= Nat.1 + c and Nat.1 + c <= n.suc
            implies (n.suc.factorial + (Nat.1 + c)).is_composite)
        (n.suc.factorial + (Nat.1 + c)).is_composite
        m.is_composite
    }
}

/// A prime above the greatest prime at most `(n+1)! + 1` is at least
/// `(n+1)! + n + 2`: the classical run of composites forces the next prime past
/// the whole run.
theorem prime_above_gap_run(n: Nat, p: Nat, q: Nat) {
    is_max(prime_at_most(n.suc.factorial + Nat.1), p) and q.is_prime and p < q
    implies n.suc.factorial + n.suc < q
} by {
    if is_max(prime_at_most(n.suc.factorial + Nat.1), p) and q.is_prime and p < q {
        max_prime_below_above(n, p, q)
        n.suc.factorial + Nat.1 < q
        if q <= n.suc.factorial + n.suc {
            factorial_shift_interval_composite(n, q)
            (n.suc.factorial + Nat.1 < q and q <= n.suc.factorial + n.suc
                implies q.is_composite)
            q.is_composite
            (q.is_prime = (Nat.1 < q and not q.is_composite))
            not q.is_composite
            false
        }
        not (q <= n.suc.factorial + n.suc)
        lt_or_lte(n.suc.factorial + n.suc, q)
        (n.suc.factorial + n.suc < q or q <= n.suc.factorial + n.suc)
        n.suc.factorial + n.suc < q
    }
}

/// There are gaps between consecutive primes longer than any given bound.
///
/// Beyond every `n`, the classical construction produces a run of `n`
/// consecutive composites; the primes immediately before and after the run are
/// consecutive primes whose gap exceeds `n`.
theorem prime_gap_unbounded(n: Nat) {
    exists(p: Nat, q: Nat) {
        p.is_prime and q.is_prime and p < q and
        (forall(m: Nat) { p < m and m < q implies not m.is_prime }) and
        p + n < q
    }
} by {
    lte_one_factorial(n.suc)
    Nat.1 <= n.suc.factorial
    lte_suc_suc(Nat.1, n.suc.factorial)
    Nat.2 <= n.suc.factorial.suc
    add_one_right(n.suc.factorial)
    (n.suc.factorial + Nat.1 = n.suc.factorial.suc)
    Nat.2 <= n.suc.factorial + Nat.1
    largest_prime_at_most_exists(n.suc.factorial + Nat.1)
    exists(m: Nat) { is_max(prime_at_most(n.suc.factorial + Nat.1), m) }
    let (p: Nat) satisfy { is_max(prime_at_most(n.suc.factorial + Nat.1), p) }
    least_prime_above_exists(p)
    exists(q2: Nat) { is_min(prime_above(p), q2) }
    let (q: Nat) satisfy { is_min(prime_above(p), q) }
    is_max_apply(prime_at_most(n.suc.factorial + Nat.1), p)
    prime_at_most(n.suc.factorial + Nat.1)(p)
    prime_at_most_eq(n.suc.factorial + Nat.1, p)
    (prime_at_most(n.suc.factorial + Nat.1)(p) =
        (p.is_prime and p <= n.suc.factorial + Nat.1))
    p.is_prime
    p <= n.suc.factorial + Nat.1
    is_min_apply(prime_above(p), q)
    prime_above(p)(q)
    prime_above_eq(p, q)
    (prime_above(p)(q) = (q.is_prime and p < q))
    q.is_prime
    p < q
    prime_above_gap_run(n, p, q)
    n.suc.factorial + n.suc < q
    lte_add_right(n, p, n.suc.factorial + Nat.1)
    p + n <= (n.suc.factorial + Nat.1) + n
    add_assoc(n.suc.factorial, Nat.1, n)
    (n.suc.factorial + Nat.1) + n = n.suc.factorial + (Nat.1 + n)
    add_comm(Nat.1, n)
    Nat.1 + n = n + Nat.1
    add_one_right(n)
    n + Nat.1 = n.suc
    n.suc.factorial + (Nat.1 + n) = n.suc.factorial + n.suc
    (n.suc.factorial + Nat.1) + n = n.suc.factorial + n.suc
    p + n <= n.suc.factorial + n.suc
    lte_and_lt(p + n, n.suc.factorial + n.suc, q)
    p + n < q
    is_min_false_below(prime_above(p), q)
    false_below(prime_above(p), q)
    (false_below(prime_above(p), q) = forall(t: Nat) {
        t < q implies not prime_above(p)(t)
    })
    forall(m: Nat) {
        if p < m and m < q {
            (m < q implies not prime_above(p)(m))
            not prime_above(p)(m)
            if m.is_prime {
                prime_above_eq(p, m)
                (prime_above(p)(m) = (m.is_prime and p < m))
                prime_above(p)(m)
                false
            }
            not m.is_prime
        }
        (p < m and m < q implies not m.is_prime)
    }
    exists(p2: Nat, q2: Nat) {
        p2.is_prime and q2.is_prime and p2 < q2 and
        (forall(m: Nat) { p2 < m and m < q2 implies not m.is_prime }) and
        p2 + n < q2
    }
}
