from nat import Nat

/// The displayed sum of four natural squares.
define four_square_sum(a: Nat, b: Nat, c: Nat, d: Nat) -> Nat {
    a * a + b * b + c * c + d * d
}

/// The helper `four_square_sum` unfolds to the displayed expression.
theorem four_square_sum_unfold(a: Nat, b: Nat, c: Nat, d: Nat) {
    four_square_sum(a, b, c, d) = a * a + b * b + c * c + d * d
}

/// A natural number is a sum of four squares if it has four natural witnesses.
define is_sum_four_squares(n: Nat) -> Bool {
    exists(a: Nat, b: Nat, c: Nat, d: Nat) {
        n = a * a + b * b + c * c + d * d
    }
}

/// Four explicit witnesses package an `is_sum_four_squares` proof.
theorem is_sum_four_squares_intro(n: Nat, a: Nat, b: Nat, c: Nat, d: Nat) {
    n = four_square_sum(a, b, c, d) implies is_sum_four_squares(n)
} by {
    if n = four_square_sum(a, b, c, d) {
        four_square_sum(a, b, c, d) = a * a + b * b + c * c + d * d
        n = a * a + b * b + c * c + d * d
        exists(x: Nat, y: Nat, z: Nat, w: Nat) {
            x = a and y = b and z = c and w = d and n = x * x + y * y + z * z + w * w
        }
    }
}

/// A four-square proof can be re-exposed using `four_square_sum` witnesses.
theorem is_sum_four_squares_elim(n: Nat) {
    is_sum_four_squares(n) implies exists(a: Nat, b: Nat, c: Nat, d: Nat) {
        n = four_square_sum(a, b, c, d)
    }
} by {
    if is_sum_four_squares(n) {
        let (a: Nat, b: Nat, c: Nat, d: Nat) satisfy {
            n = a * a + b * b + c * c + d * d
        }
        four_square_sum(a, b, c, d) = a * a + b * b + c * c + d * d
        n = four_square_sum(a, b, c, d)
        exists(x: Nat, y: Nat, z: Nat, w: Nat) {
            x = a and y = b and z = c and w = d and n = four_square_sum(x, y, z, w)
        }
    }
}

/// Equality transports the four-square predicate.
theorem is_sum_four_squares_of_eq(n: Nat, m: Nat) {
    n = m and is_sum_four_squares(n) implies is_sum_four_squares(m)
}

/// Every explicit four-square sum satisfies the predicate.
theorem four_square_sum_is_sum_four_squares(a: Nat, b: Nat, c: Nat, d: Nat) {
    is_sum_four_squares(four_square_sum(a, b, c, d))
} by {
    is_sum_four_squares_intro(four_square_sum(a, b, c, d), a, b, c, d)
}

/// Zero is a sum of four squares.
theorem is_sum_four_squares_zero {
    is_sum_four_squares(Nat.0)
} by {
    Nat.0 = Nat.0 * Nat.0 + Nat.0 * Nat.0 + Nat.0 * Nat.0 + Nat.0 * Nat.0
    exists(a: Nat, b: Nat, c: Nat, d: Nat) {
        Nat.0 = a * a + b * b + c * c + d * d
    }
}

/// One is a sum of four squares.
theorem is_sum_four_squares_one {
    is_sum_four_squares(Nat.1)
} by {
    Nat.1 = Nat.1 * Nat.1 + Nat.0 * Nat.0 + Nat.0 * Nat.0 + Nat.0 * Nat.0
    exists(a: Nat, b: Nat, c: Nat, d: Nat) {
        Nat.1 = a * a + b * b + c * c + d * d
    }
}

/// Two is a sum of four squares.
theorem is_sum_four_squares_two {
    is_sum_four_squares(Nat.2)
} by {
    Nat.2 = Nat.1 * Nat.1 + Nat.1 * Nat.1 + Nat.0 * Nat.0 + Nat.0 * Nat.0
    exists(a: Nat, b: Nat, c: Nat, d: Nat) {
        Nat.2 = a * a + b * b + c * c + d * d
    }
}

/// Three is a sum of four squares.
theorem is_sum_four_squares_three {
    is_sum_four_squares(Nat.3)
} by {
    Nat.3 = Nat.1 * Nat.1 + Nat.1 * Nat.1 + Nat.1 * Nat.1 + Nat.0 * Nat.0
    exists(a: Nat, b: Nat, c: Nat, d: Nat) {
        Nat.3 = a * a + b * b + c * c + d * d
    }
}

/// Four is a sum of four squares.
theorem is_sum_four_squares_four {
    is_sum_four_squares(Nat.4)
} by {
    Nat.4 = Nat.1 * Nat.1 + Nat.1 * Nat.1 + Nat.1 * Nat.1 + Nat.1 * Nat.1
    exists(a: Nat, b: Nat, c: Nat, d: Nat) {
        Nat.4 = a * a + b * b + c * c + d * d
    }
}

/// Every square is a sum of four squares.
theorem is_sum_four_squares_square(x: Nat) {
    is_sum_four_squares(x * x)
} by {
    x * x = x * x + Nat.0 * Nat.0 + Nat.0 * Nat.0 + Nat.0 * Nat.0
    exists(a: Nat, b: Nat, c: Nat, d: Nat) {
        x * x = a * a + b * b + c * c + d * d
    }
}

/// The first coordinate gives a four-square representation of a square.
theorem is_sum_four_squares_single_first(x: Nat) {
    is_sum_four_squares(four_square_sum(x, Nat.0, Nat.0, Nat.0))
} by {
    four_square_sum_is_sum_four_squares(x, Nat.0, Nat.0, Nat.0)
}

/// The second coordinate gives a four-square representation of a square.
theorem is_sum_four_squares_single_second(x: Nat) {
    is_sum_four_squares(four_square_sum(Nat.0, x, Nat.0, Nat.0))
} by {
    four_square_sum_is_sum_four_squares(Nat.0, x, Nat.0, Nat.0)
}

/// The third coordinate gives a four-square representation of a square.
theorem is_sum_four_squares_single_third(x: Nat) {
    is_sum_four_squares(four_square_sum(Nat.0, Nat.0, x, Nat.0))
} by {
    four_square_sum_is_sum_four_squares(Nat.0, Nat.0, x, Nat.0)
}

/// The fourth coordinate gives a four-square representation of a square.
theorem is_sum_four_squares_single_fourth(x: Nat) {
    is_sum_four_squares(four_square_sum(Nat.0, Nat.0, Nat.0, x))
} by {
    four_square_sum_is_sum_four_squares(Nat.0, Nat.0, Nat.0, x)
}

/// Two squares are a degenerate sum of four squares.
theorem is_sum_four_squares_two_squares(a: Nat, b: Nat) {
    is_sum_four_squares(a * a + b * b)
} by {
    a * a + b * b = a * a + b * b + Nat.0 * Nat.0 + Nat.0 * Nat.0
    exists(x: Nat, y: Nat, z: Nat, w: Nat) {
        a * a + b * b = x * x + y * y + z * z + w * w
    }
}

/// Three squares are a degenerate sum of four squares.
theorem is_sum_four_squares_three_squares(a: Nat, b: Nat, c: Nat) {
    is_sum_four_squares(a * a + b * b + c * c)
} by {
    a * a + b * b + c * c = a * a + b * b + c * c + Nat.0 * Nat.0
    exists(x: Nat, y: Nat, z: Nat, w: Nat) {
        a * a + b * b + c * c = x * x + y * y + z * z + w * w
    }
}

/// Multiplication preserves the square of a natural product.
theorem square_mul_square(k: Nat, a: Nat) {
    (k * a) * (k * a) = k * k * (a * a)
} by {
    (k * a) * (k * a) = ((k * a) * k) * a
    (k * a) * k = k * (a * k)
    a * k = k * a
    k * (a * k) = k * (k * a)
    k * (k * a) = (k * k) * a
    (k * a) * k = (k * k) * a
    ((k * a) * k) * a = ((k * k) * a) * a
    ((k * k) * a) * a = (k * k) * (a * a)
    k * k * (a * a) = (k * k) * (a * a)
}

/// Multiplication by a square distributes over a sum of four squares.
theorem mul_square_distrib_four(k: Nat, a: Nat, b: Nat, c: Nat, d: Nat) {
    k * k * (a * a + b * b + c * c + d * d) =
    (k * a) * (k * a) + (k * b) * (k * b) +
    (k * c) * (k * c) + (k * d) * (k * d)
} by {
    k * k * (a * a + b * b + c * c + d * d) =
        k * k * (a * a + b * b + c * c) + k * k * (d * d)
    k * k * (a * a + b * b + c * c) =
        k * k * (a * a + b * b) + k * k * (c * c)
    k * k * (a * a + b * b) = k * k * (a * a) + k * k * (b * b)
    k * k * (a * a) = (k * a) * (k * a)
    k * k * (b * b) = (k * b) * (k * b)
    k * k * (c * c) = (k * c) * (k * c)
    k * k * (d * d) = (k * d) * (k * d)
    k * k * (a * a + b * b) = (k * a) * (k * a) + (k * b) * (k * b)
    k * k * (a * a + b * b + c * c) =
        (k * a) * (k * a) + (k * b) * (k * b) + (k * c) * (k * c)
    k * k * (a * a + b * b + c * c + d * d) =
        (k * a) * (k * a) + (k * b) * (k * b) +
        (k * c) * (k * c) + (k * d) * (k * d)
}

/// Scaling all four witnesses by the same factor scales their four-square sum by a square.
theorem four_square_sum_scale(k: Nat, a: Nat, b: Nat, c: Nat, d: Nat) {
    four_square_sum(k * a, k * b, k * c, k * d) = k * k * four_square_sum(a, b, c, d)
} by {
    four_square_sum(k * a, k * b, k * c, k * d) =
        (k * a) * (k * a) + (k * b) * (k * b) +
        (k * c) * (k * c) + (k * d) * (k * d)
    k * k * four_square_sum(a, b, c, d) = k * k * (a * a + b * b + c * c + d * d)
    k * k * (a * a + b * b + c * c + d * d) =
        (k * a) * (k * a) + (k * b) * (k * b) +
        (k * c) * (k * c) + (k * d) * (k * d)
    four_square_sum(k * a, k * b, k * c, k * d) = k * k * four_square_sum(a, b, c, d)
}

/// Closure under multiplication by a square.
/// This is a staged helper toward the four-square theorem, not the full theorem.
theorem is_sum_four_squares_mul_square(n: Nat, k: Nat) {
    is_sum_four_squares(n) implies is_sum_four_squares(k * k * n)
} by {
    let (a: Nat, b: Nat, c: Nat, d: Nat) satisfy {
        n = a * a + b * b + c * c + d * d
    }
    k * k * n = k * k * (a * a + b * b + c * c + d * d)
    k * k * (a * a + b * b + c * c + d * d) =
        (k * a) * (k * a) + (k * b) * (k * b) +
        (k * c) * (k * c) + (k * d) * (k * d)
    k * k * n =
        (k * a) * (k * a) + (k * b) * (k * b) +
        (k * c) * (k * c) + (k * d) * (k * d)
    exists(x: Nat, y: Nat, z: Nat, w: Nat) {
        k * k * n = x * x + y * y + z * z + w * w
    }
}

/// Closure under right multiplication by a square, with the square factor on the right.
theorem is_sum_four_squares_mul_square_right(n: Nat, k: Nat) {
    is_sum_four_squares(n) implies is_sum_four_squares(n * (k * k))
} by {
    if is_sum_four_squares(n) {
        n * (k * k) = k * k * n
        is_sum_four_squares_mul_square(n, k)
        is_sum_four_squares(n * (k * k))
    }
}

/// Closure under multiplication by the square `k * k` named as a product.
theorem is_sum_four_squares_square_mul(n: Nat, k: Nat) {
    is_sum_four_squares(n) implies is_sum_four_squares((k * k) * n)
} by {
    is_sum_four_squares_mul_square(n, k)
}

/// Multiplying by one square leaves a four-square representation available.
theorem is_sum_four_squares_mul_one_square(n: Nat) {
    is_sum_four_squares(n) implies is_sum_four_squares(Nat.1 * Nat.1 * n)
} by {
    is_sum_four_squares_mul_square(n, Nat.1)
}

/// Multiplying zero by any square remains a sum of four squares.
theorem is_sum_four_squares_zero_mul_square(k: Nat) {
    is_sum_four_squares(k * k * Nat.0)
} by {
    k * k * Nat.0 = Nat.0
    is_sum_four_squares_zero
    is_sum_four_squares(k * k * Nat.0)
}

/// The square of a product is a sum of four squares.
theorem is_sum_four_squares_product_square(k: Nat, a: Nat) {
    is_sum_four_squares((k * a) * (k * a))
} by {
    is_sum_four_squares_square(k * a)
}
