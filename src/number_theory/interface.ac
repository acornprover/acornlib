/// Public interface for number theory.

from nat import Nat, gcd_pair, gcd_step_n, num_gcd_steps

// coprime.ac
from nat import gcd_comm, gcd_one_left, gcd_one_right, gcd_mult_right, gcd_divides, gcd_divides_left,
    gcd_divides_right, divides_gcd
from nat import divides_mul, divides_mod, divides_unmod

numerals Nat

/// One is coprime with every natural number on the left.
theorem coprime_one_left(a: Nat) {
    Nat.1.coprime(a)
}

/// If zero is coprime to a natural number, that number is one.
theorem coprime_zero_left_imp_one(a: Nat) {
    Nat.0.coprime(a) implies a = Nat.1
}

/// If a natural number is coprime to zero, that number is one.
theorem coprime_zero_right_imp_one(a: Nat) {
    a.coprime(Nat.0) implies a = Nat.1
}

/// Any divisor of one is one.
theorem nat_divides_one_imp_one(d: Nat) {
    d.divides(Nat.1) implies d = Nat.1
}

/// Divisors of coprime natural numbers are coprime.
theorem coprime_of_divisors(a: Nat, b: Nat, d: Nat, e: Nat) {
    a.coprime(b) and d.divides(a) and e.divides(b) implies d.coprime(e)
}


// factorisation.ac
from nat import gcd_of_prime, gcd_zero_left, gcd_zero_right, gcd_nonzero_left, gcd_nonzero_right
from nat import gcd_mul_lcm
from nat import exp_zero, exp_ne_zero
from nat import divides_self, divides_zero, has_prime_divisor, strong_induction, true_below,
    divides_trans, mul_cancel_left, mul_to_zero, divides_mul_left, add_cancels_left
from list import List, list_not_contains_impl_count_zero
from list import product
from list import map, sum
from list import is_permutation, remove_one_count_self, remove_one_count_other, remove_one_cons_eq,
    remove_one_cons_neq, nil_count_zero, product_remove_one, count_append, product_append,
    permutation_preserves_product

/// Distinct primes are coprime.
theorem coprime_of_distinct_primes(p: Nat, q: Nat) {
    p.is_prime and q.is_prime and p != q implies p.coprime(q)
}

/// True if every element of the list is a prime natural number.
define all_prime(list: List[Nat]) -> Bool {
    match list {
        List.nil {
            true
        }
        List.cons(head, tail) {
            head.is_prime and all_prime(tail)
        }
    }
}

/// Existence of a prime factorisation: every positive natural is the product
/// of some list of primes.
theorem prime_factorisation_exists(n: Nat) {
    Nat.1 <= n implies exists(factors: List[Nat]) {
        product[Nat](factors) = n and all_prime(factors)
    }
}

/// A canonical prime factorisation: a list of primes whose product is the
/// given natural. For zero the result is the empty list (a placeholder; the
/// universal property only constrains positive inputs).
let prime_factorisation(n: Nat) -> factors: List[Nat] satisfy {
    if Nat.1 <= n {
        product[Nat](factors) = n and all_prime(factors)
    } else {
        factors = List.nil[Nat]
    }
}

/// Forward direction (packaged): a prime has no proper divisor strictly between 1 and itself.
theorem prime_imp_no_proper_divisor(n: Nat) {
    n.is_prime implies Nat.1 < n and forall(k: Nat) {
        Nat.1 < k and k < n implies not k.divides(n)
    }
}

/// Backward direction: if n > 1 and no number strictly between 1 and n divides n, then n is prime.
theorem no_proper_divisor_imp_prime(n: Nat) {
    Nat.1 < n and (forall(k: Nat) {
        Nat.1 < k and k < n implies not k.divides(n)
    }) implies n.is_prime
}

/// Uniqueness side of the fundamental theorem of arithmetic: any two prime
/// factorisations of the same natural number are permutations of each other.
theorem prime_factorisation_unique(l1: List[Nat], l2: List[Nat]) {
    all_prime(l1) and all_prime(l2) and product[Nat](l1) = product[Nat](l2)
        implies is_permutation(l1, l2)
}

/// The multiplicity of a prime in the canonical factorisation of a natural.
/// For non-primes, for one, and for zero, this evaluates to zero (the factorisation
/// of zero is the empty placeholder).
define count_prime_factor(p: Nat, n: Nat) -> Nat {
    prime_factorisation(n).count(p)
}

from nat import carry_count_fuel, digit_sum, addition_carry_count,
    double_addition_carry_count, is_addition_carry_count
from combinatorics import binom

// legendre.ac
/// The running total of the p-adic valuations of `1, 2, ..., n`, where the
/// valuation of `m` is `count_prime_factor(p, m)`.
define prime_factor_count_upto(p: Nat, n: Nat) -> Nat {
    match n {
        Nat.zero {
            Nat.0
        }
        Nat.suc(k) {
            prime_factor_count_upto(p, k) + count_prime_factor(p, n)
        }
    }
}

/// The empty initial segment contributes nothing.
theorem prime_factor_count_upto_zero(p: Nat) {
    prime_factor_count_upto(p, Nat.0) = Nat.0
}

/// Extending the range by one adds the valuation of the new top element.
theorem prime_factor_count_upto_step(p: Nat, n: Nat) {
    prime_factor_count_upto(p, n.suc) =
        prime_factor_count_upto(p, n) + count_prime_factor(p, n.suc)
}

/// The factorial is positive, hence nonzero.
theorem factorial_ne_zero(n: Nat) {
    n.factorial != Nat.0
}

/// Legendre's identity in additive form.
theorem legendre_factorial(p: Nat, n: Nat) {
    count_prime_factor(p, n.factorial) = prime_factor_count_upto(p, n)
}

/// A product of nonzero naturals is nonzero.
theorem mul_ne_zero(a: Nat, b: Nat) {
    a != Nat.0 and b != Nat.0 implies a * b != Nat.0
}

/// The binomial coefficient `binom(a+b, a)` is nonzero.
theorem binom_add_ne_zero(a: Nat, b: Nat) {
    (a + b).binom(a) != Nat.0
}

/// The p-adic valuation accounting for `binom(a+b,a)` in factorial form.
theorem legendre_binom_factorial(p: Nat, a: Nat, b: Nat) {
    count_prime_factor(p, (a + b).binom(a)) +
        count_prime_factor(p, a.factorial) +
        count_prime_factor(p, b.factorial) =
        count_prime_factor(p, (a + b).factorial)
}

/// Legendre's identity for binomial coefficients, additive form.
theorem legendre_binom(p: Nat, a: Nat, b: Nat) {
    count_prime_factor(p, (a + b).binom(a)) +
        prime_factor_count_upto(p, a) +
        prime_factor_count_upto(p, b) =
        prime_factor_count_upto(p, a + b)
}

/// For naturals, being at least one is the same as being nonzero.
theorem one_lte_iff_ne_zero(x: Nat) {
    (Nat.1 <= x) = (x != Nat.0)
}

/// A prime `p` divides nonzero `n` exactly when its valuation is nonzero.
theorem prime_divides_iff_count_ne_zero(p: Nat, n: Nat) {
    p.is_prime and n != Nat.0 implies (
        p.divides(n) = (count_prime_factor(p, n) != Nat.0)
    )
}

/// A prime divides `binom(a+b, a)` exactly when its valuation there is nonzero.
theorem prime_divides_binom_iff(p: Nat, a: Nat, b: Nat) {
    p.is_prime implies (
        p.divides((a + b).binom(a)) =
            (count_prime_factor(p, (a + b).binom(a)) != Nat.0)
    )
}

// legendre_recurrence.ac
/// Legendre's recurrence for the running valuation total.
theorem legendre_recurrence(p: Nat, n: Nat) {
    p.is_prime implies
        prime_factor_count_upto(p, n) =
            n.div(p) + prime_factor_count_upto(p, n.div(p))
}

/// Legendre's recurrence stated directly for the valuation of `n!`.
theorem legendre_factorial_recurrence(p: Nat, n: Nat) {
    p.is_prime implies
        count_prime_factor(p, n.factorial) =
            n.div(p) + count_prime_factor(p, n.div(p).factorial)
}

// kummer.ac
/// The digit-sum form of Legendre's formula at a single natural number.
define legendre_digit_sum_at(p: Nat, n: Nat) -> Bool {
    p * count_prime_factor(p, n.factorial) + digit_sum(p, n) =
        n + count_prime_factor(p, n.factorial)
}

/// The digit-sum form of Legendre's formula.
theorem legendre_digit_sum(p: Nat, n: Nat) {
    p.is_prime implies legendre_digit_sum_at(p, n)
}

/// The digit-sum form of Legendre's formula at `a`, `b`, and `a+b`
/// implies Kummer's digit-sum identity for `binom(a+b,a)`.
theorem kummer_digit_sum_of_legendre_digit_sum(p: Nat, a: Nat, b: Nat) {
    legendre_digit_sum_at(p, a) and
    legendre_digit_sum_at(p, b) and
    legendre_digit_sum_at(p, a + b)
        implies
    p * count_prime_factor(p, (a + b).binom(a)) + digit_sum(p, a + b) =
        count_prime_factor(p, (a + b).binom(a)) + digit_sum(p, a) + digit_sum(p, b)
}

/// Kummer's theorem in additive digit-sum form: for a prime `p`, the p-adic
/// valuation of `binom(a+b,a)` satisfies the digit-sum carry equation, so the
/// valuation equals the number of carries when adding `a` and `b` in base `p`.
theorem kummer_digit_sum(p: Nat, a: Nat, b: Nat) {
    p.is_prime implies
        p * count_prime_factor(p, (a + b).binom(a)) + digit_sum(p, a + b) =
            count_prime_factor(p, (a + b).binom(a)) + digit_sum(p, a) + digit_sum(p, b)
}

// falling_product.ac
/// The central binomial coefficient.
define central_binom(n: Nat) -> Nat {
    (n + n).binom(n)
}

/// The `i`th factor in the falling product from `n`.
define falling_product_factor(n: Nat, i: Nat) -> Nat {
    n - i
}

/// The prime multiplicity of the `i`th factor in the falling product from `n`.
define falling_product_factor_count(p: Nat, n: Nat, i: Nat) -> Nat {
    count_prime_factor(p, falling_product_factor(n, i))
}

/// The factors `n, n - 1, ..., n - k` as a list.
define falling_product_factors(n: Nat, k: Nat) -> List[Nat] {
    map(k.suc.range, falling_product_factor(n))
}

/// The prime multiplicities of the factors `n, n - 1, ..., n - k` as a list.
define falling_product_factor_counts(p: Nat, n: Nat, k: Nat) -> List[Nat] {
    map(k.suc.range, falling_product_factor_count(p, n))
}

/// The product `n * (n - 1) * ... * (n - k)`.
define falling_product(n: Nat, k: Nat) -> Nat {
    match k {
        Nat.zero {
            n
        }
        Nat.suc(j) {
            falling_product(n, j) * (n - k)
        }
    }
}

/// The sum of prime multiplicities in the factors of a falling product.
define falling_product_prime_count_sum(p: Nat, n: Nat, k: Nat) -> Nat {
    match k {
        Nat.zero {
            count_prime_factor(p, n)
        }
        Nat.suc(j) {
            falling_product_prime_count_sum(p, n, j) +
                count_prime_factor(p, n - k)
        }
    }
}

/// The central binomial coefficient unfolds to its defining binomial coefficient.
theorem central_binom_eq(n: Nat) {
    central_binom(n) = (n + n).binom(n)
}

/// The zeroth central binomial coefficient is one.
theorem central_binom_zero {
    central_binom(Nat.0) = Nat.1
}

/// The first central binomial coefficient is two.
theorem central_binom_one {
    central_binom(Nat.1) = Nat.2
}

/// The central binomial coefficient is nonzero.
theorem central_binom_ne_zero(n: Nat) {
    central_binom(n) != Nat.0
}

/// The zeroth falling product is the top factor.
theorem falling_product_zero(n: Nat) {
    falling_product(n, Nat.0) = n
}

/// Extending a falling product appends the next lower factor.
theorem falling_product_suc(n: Nat, k: Nat) {
    falling_product(n, k.suc) = falling_product(n, k) * (n - k.suc)
}

/// The zeroth falling-product factor list is the singleton list containing `n`.
theorem falling_product_factors_zero(n: Nat) {
    falling_product_factors(n, Nat.0) = List.singleton(n)
}

/// Extending the factor list appends the next lower factor.
theorem falling_product_factors_suc(n: Nat, k: Nat) {
    falling_product_factors(n, k.suc) =
        falling_product_factors(n, k).append(n - k.suc)
}

/// The zeroth falling-product factor-count list is the singleton list
/// containing the valuation of `n`.
theorem falling_product_factor_counts_zero(p: Nat, n: Nat) {
    falling_product_factor_counts(p, n, Nat.0) =
        List.singleton(count_prime_factor(p, n))
}

/// Extending the factor-count list appends the valuation of the next lower
/// factor.
theorem falling_product_factor_counts_suc(p: Nat, n: Nat, k: Nat) {
    falling_product_factor_counts(p, n, k.suc) =
        falling_product_factor_counts(p, n, k).append(count_prime_factor(p, n - k.suc))
}

/// The zeroth falling-product valuation sum is the valuation of the top factor.
theorem falling_product_prime_count_sum_zero(p: Nat, n: Nat) {
    falling_product_prime_count_sum(p, n, Nat.0) = count_prime_factor(p, n)
}

/// Extending the valuation sum appends the valuation of the next factor.
theorem falling_product_prime_count_sum_suc(p: Nat, n: Nat, k: Nat) {
    falling_product_prime_count_sum(p, n, k.suc) =
        falling_product_prime_count_sum(p, n, k) + count_prime_factor(p, n - k.suc)
}

/// The recursive falling product is the product of its factor list.
theorem falling_product_eq_product_factors(n: Nat, k: Nat) {
    falling_product(n, k) = product[Nat](falling_product_factors(n, k))
}

/// The recursive falling-product valuation sum is the sum of its
/// factor-count list.
theorem falling_product_prime_count_sum_eq_sum_factor_counts(p: Nat, n: Nat, k: Nat) {
    falling_product_prime_count_sum(p, n, k) =
        sum[Nat](falling_product_factor_counts(p, n, k))
}





/// The factor list for the falling product with two factors.
theorem falling_product_factors_one(n: Nat) {
    falling_product_factors(n, Nat.1) =
        List.cons(n, List.cons(n - Nat.1, List.nil[Nat]))
}

/// The factor list for the falling product with three factors.
theorem falling_product_factors_two(n: Nat) {
    falling_product_factors(n, Nat.2) =
        List.cons(n, List.cons(n - Nat.1, List.cons(n - Nat.2, List.nil[Nat])))
}

/// The factor-count list for the falling product with two factors.
theorem falling_product_factor_counts_one(p: Nat, n: Nat) {
    falling_product_factor_counts(p, n, Nat.1) =
        List.cons(count_prime_factor(p, n),
            List.cons(count_prime_factor(p, n - Nat.1), List.nil[Nat]))
}

/// The factor-count list for the falling product with three factors.
theorem falling_product_factor_counts_two(p: Nat, n: Nat) {
    falling_product_factor_counts(p, n, Nat.2) =
        List.cons(count_prime_factor(p, n),
            List.cons(count_prime_factor(p, n - Nat.1),
                List.cons(count_prime_factor(p, n - Nat.2), List.nil[Nat])))
}

/// The valuation sum for the falling product with two factors.
theorem falling_product_prime_count_sum_one(p: Nat, n: Nat) {
    falling_product_prime_count_sum(p, n, Nat.1) =
        count_prime_factor(p, n) + count_prime_factor(p, n - Nat.1)
}

/// The valuation sum for the falling product with three factors.
theorem falling_product_prime_count_sum_two(p: Nat, n: Nat) {
    falling_product_prime_count_sum(p, n, Nat.2) =
        count_prime_factor(p, n) +
        count_prime_factor(p, n - Nat.1) +
        count_prime_factor(p, n - Nat.2)
}

/// A falling product with all factors strictly positive is nonzero.
theorem falling_product_nonzero(n: Nat, k: Nat) {
    k < n implies falling_product(n, k) != Nat.0
}

/// The prime multiplicity of a falling product is the sum of the multiplicities
/// of its factors.
theorem count_prime_factor_falling_product(p: Nat, n: Nat, k: Nat) {
    k < n implies
        count_prime_factor(p, falling_product(n, k)) =
        falling_product_prime_count_sum(p, n, k)
}

/// A falling product is the initial quotient of `n!` by the remaining
/// factorial.
theorem falling_product_mul_factorial_complement(n: Nat, k: Nat) {
    k < n implies
        falling_product(n, k) * (n - k.suc).factorial = n.factorial
}

/// Every positive falling product from `n` divides `n!`.
theorem falling_product_divides_factorial(n: Nat, k: Nat) {
    k < n implies falling_product(n, k).divides(n.factorial)
}

/// Each falling product divides itself.
theorem falling_product_divides_self(n: Nat, k: Nat) {
    falling_product(n, k).divides(falling_product(n, k))
}

/// A falling product divides the product with one more factor.
theorem falling_product_divides_suc(n: Nat, k: Nat) {
    falling_product(n, k).divides(falling_product(n, k.suc))
}

/// Shorter falling products divide longer falling products.
theorem falling_product_prefix_divides(n: Nat, j: Nat, k: Nat) {
    j <= k implies falling_product(n, j).divides(falling_product(n, k))
}

/// Legendre's binomial valuation identity for central binomial coefficients.
theorem central_binom_legendre(p: Nat, n: Nat) {
    count_prime_factor(p, central_binom(n)) +
        prime_factor_count_upto(p, n) +
        prime_factor_count_upto(p, n) =
        prime_factor_count_upto(p, n + n)
}

/// Kummer's digit-sum identity for central binomial coefficients.
theorem central_binom_kummer_digit_sum(p: Nat, n: Nat) {
    p.is_prime implies
    p * count_prime_factor(p, central_binom(n)) + digit_sum(p, n + n) =
        count_prime_factor(p, central_binom(n)) + digit_sum(p, n) + digit_sum(p, n)
}

/// Two is prime.
theorem nat_two_prime {
    Nat.2.is_prime
}

/// The 2-adic valuation of the central binomial coefficient is the binary
/// digit sum of `n`.
theorem central_binom_two_adic_valuation(n: Nat) {
    count_prime_factor(Nat.2, central_binom(n)) = digit_sum(Nat.2, n)
}

// kummer_carry.ac
/// The p-adic valuation of `binom(a+b,a)` satisfies the digit-sum equation
/// characterizing the carry count for adding `a` and `b` in base `p`.
theorem binom_valuation_is_addition_carry_count(p: Nat, a: Nat, b: Nat) {
    p.is_prime implies
        is_addition_carry_count(p, a, b,
            count_prime_factor(p, (a + b).binom(a)))
}

/// The p-adic valuation of the central binomial coefficient satisfies the
/// carry-count digit-sum equation for doubling `n` in base `p`.
theorem central_binom_valuation_is_addition_carry_count(p: Nat, n: Nat) {
    p.is_prime implies
        is_addition_carry_count(p, n, n,
            count_prime_factor(p, central_binom(n)))
}

/// The p-adic valuation of `binom(a+b,a)` is the recursive carry count for
/// adding `a` and `b` in base `p`.
theorem binom_valuation_eq_addition_carry_count(p: Nat, a: Nat, b: Nat) {
    p.is_prime implies
        count_prime_factor(p, (a + b).binom(a)) =
            addition_carry_count(p, a, b)
}

/// The recursive carry count for adding `a` and `b` in base `p` is the
/// p-adic valuation of `binom(a+b,a)`.
theorem addition_carry_count_eq_binom_valuation(p: Nat, a: Nat, b: Nat) {
    p.is_prime implies
        addition_carry_count(p, a, b) =
            count_prime_factor(p, (a + b).binom(a))
}

/// The p-adic valuation of `binom(a+b,b)` is the recursive carry count for
/// adding `a` and `b` in base `p`.
theorem symmetric_binom_valuation_eq_addition_carry_count(
    p: Nat, a: Nat, b: Nat
) {
    p.is_prime implies
        count_prime_factor(p, (a + b).binom(b)) =
            addition_carry_count(p, a, b)
}

/// The recursive carry count for adding `a` and `b` in base `p` is the
/// p-adic valuation of `binom(a+b,b)`.
theorem addition_carry_count_eq_symmetric_binom_valuation(
    p: Nat, a: Nat, b: Nat
) {
    p.is_prime implies
        addition_carry_count(p, a, b) =
            count_prime_factor(p, (a + b).binom(b))
}

/// Any sufficiently large zero-incoming fuel computes the p-adic valuation of
/// `binom(a+b,a)`.
theorem carry_count_fuel_eq_binom_valuation_of_sum_le_fuel(
    p: Nat, a: Nat, b: Nat, fuel: Nat
) {
    p.is_prime and a + b <= fuel implies
        carry_count_fuel(p, a, b, Nat.0, fuel) =
            count_prime_factor(p, (a + b).binom(a))
}

/// The p-adic valuation of `binom(a+b,a)` is computed by any sufficiently
/// large zero-incoming fuel.
theorem binom_valuation_eq_carry_count_fuel_of_sum_le_fuel(
    p: Nat, a: Nat, b: Nat, fuel: Nat
) {
    p.is_prime and a + b <= fuel implies
        count_prime_factor(p, (a + b).binom(a)) =
            carry_count_fuel(p, a, b, Nat.0, fuel)
}

/// The canonical fuel `a+b` computes the p-adic valuation of
/// `binom(a+b,a)`.
theorem carry_count_fuel_at_sum_eq_binom_valuation(
    p: Nat, a: Nat, b: Nat
) {
    p.is_prime implies
        carry_count_fuel(p, a, b, Nat.0, a + b) =
            count_prime_factor(p, (a + b).binom(a))
}

/// The p-adic valuation of `binom(a+b,a)` is computed by the canonical fuel
/// `a+b`.
theorem binom_valuation_eq_carry_count_fuel_at_sum(
    p: Nat, a: Nat, b: Nat
) {
    p.is_prime implies
        count_prime_factor(p, (a + b).binom(a)) =
            carry_count_fuel(p, a, b, Nat.0, a + b)
}

/// The canonical fuel `a+b` computes the p-adic valuation of
/// `binom(a+b,b)`.
theorem carry_count_fuel_at_sum_eq_symmetric_binom_valuation(
    p: Nat, a: Nat, b: Nat
) {
    p.is_prime implies
        carry_count_fuel(p, a, b, Nat.0, a + b) =
            count_prime_factor(p, (a + b).binom(b))
}

/// The p-adic valuation of `binom(a+b,b)` is computed by the canonical fuel
/// `a+b`.
theorem symmetric_binom_valuation_eq_carry_count_fuel_at_sum(
    p: Nat, a: Nat, b: Nat
) {
    p.is_prime implies
        count_prime_factor(p, (a + b).binom(b)) =
            carry_count_fuel(p, a, b, Nat.0, a + b)
}

/// The p-adic valuation of the central binomial coefficient is the recursive
/// carry count for adding `n` to itself in base `p`.
theorem central_binom_valuation_eq_addition_carry_count(p: Nat, n: Nat) {
    p.is_prime implies
        count_prime_factor(p, central_binom(n)) =
            addition_carry_count(p, n, n)
}

/// The recursive carry count for adding `n` to itself in base `p` is the
/// p-adic valuation of the central binomial coefficient.
theorem addition_carry_count_eq_central_binom_valuation(p: Nat, n: Nat) {
    p.is_prime implies
        addition_carry_count(p, n, n) =
            count_prime_factor(p, central_binom(n))
}

/// The p-adic valuation of the central binomial coefficient is the
/// doubled-addend recursive carry count in base `p`.
theorem central_binom_valuation_eq_double_addition_carry_count(p: Nat, n: Nat) {
    p.is_prime implies
        count_prime_factor(p, central_binom(n)) =
            double_addition_carry_count(p, n)
}

/// The doubled-addend recursive carry count in base `p` is the p-adic
/// valuation of the central binomial coefficient.
theorem double_addition_carry_count_eq_central_binom_valuation(p: Nat, n: Nat) {
    p.is_prime implies
        double_addition_carry_count(p, n) =
            count_prime_factor(p, central_binom(n))
}

/// Any sufficiently large fuel for doubling computes the p-adic valuation of
/// the central binomial coefficient.
theorem carry_count_fuel_eq_central_binom_valuation_of_double_le_fuel(
    p: Nat, n: Nat, fuel: Nat
) {
    p.is_prime and n + n <= fuel implies
        carry_count_fuel(p, n, n, Nat.0, fuel) =
            count_prime_factor(p, central_binom(n))
}

/// The p-adic valuation of the central binomial coefficient is computed by
/// any sufficiently large fuel for doubling.
theorem central_binom_valuation_eq_carry_count_fuel_of_double_le_fuel(
    p: Nat, n: Nat, fuel: Nat
) {
    p.is_prime and n + n <= fuel implies
        count_prime_factor(p, central_binom(n)) =
            carry_count_fuel(p, n, n, Nat.0, fuel)
}

/// The canonical doubling fuel `n+n` computes the p-adic valuation of the
/// central binomial coefficient.
theorem carry_count_fuel_at_double_sum_eq_central_binom_valuation(
    p: Nat, n: Nat
) {
    p.is_prime implies
        carry_count_fuel(p, n, n, Nat.0, n + n) =
            count_prime_factor(p, central_binom(n))
}

/// The p-adic valuation of the central binomial coefficient is computed by
/// the canonical doubling fuel `n+n`.
theorem central_binom_valuation_eq_carry_count_fuel_at_double_sum(
    p: Nat, n: Nat
) {
    p.is_prime implies
        count_prime_factor(p, central_binom(n)) =
            carry_count_fuel(p, n, n, Nat.0, n + n)
}

/// The binary doubled-addend carry count is the binary digit sum.
theorem double_addition_carry_count_two_eq_digit_sum(n: Nat) {
    double_addition_carry_count(Nat.2, n) = digit_sum(Nat.2, n)
}

/// The binary digit sum is the binary doubled-addend carry count.
theorem digit_sum_eq_double_addition_carry_count_two(n: Nat) {
    digit_sum(Nat.2, n) = double_addition_carry_count(Nat.2, n)
}

/// The binary recursive carry count for adding `n` to itself is the binary
/// digit sum.
theorem addition_carry_count_two_double_eq_digit_sum(n: Nat) {
    addition_carry_count(Nat.2, n, n) = digit_sum(Nat.2, n)
}

/// Any sufficiently large binary fuel for doubling computes the binary digit
/// sum.
theorem carry_count_fuel_two_double_eq_digit_sum_of_double_le_fuel(
    n: Nat, fuel: Nat
) {
    n + n <= fuel implies
        carry_count_fuel(Nat.2, n, n, Nat.0, fuel) = digit_sum(Nat.2, n)
}

/// The binary digit sum is computed by any sufficiently large binary fuel for
/// doubling.
theorem digit_sum_eq_carry_count_fuel_two_double_of_double_le_fuel(
    n: Nat, fuel: Nat
) {
    n + n <= fuel implies
        digit_sum(Nat.2, n) = carry_count_fuel(Nat.2, n, n, Nat.0, fuel)
}

/// The canonical binary doubling fuel computes the binary digit sum.
theorem carry_count_fuel_two_double_at_double_sum_eq_digit_sum(n: Nat) {
    carry_count_fuel(Nat.2, n, n, Nat.0, n + n) = digit_sum(Nat.2, n)
}

/// The binary digit sum is computed by the canonical binary doubling fuel.
theorem digit_sum_eq_carry_count_fuel_two_double_at_double_sum(n: Nat) {
    digit_sum(Nat.2, n) = carry_count_fuel(Nat.2, n, n, Nat.0, n + n)
}

// lcm.ac
/// Zero is absorbing for lcm on the left.
theorem lcm_zero_left(a: Nat) {
    Nat.0.lcm(a) = Nat.0
}

/// Zero is absorbing for lcm on the right.
theorem lcm_zero_right(a: Nat) {
    a.lcm(Nat.0) = Nat.0
}

/// The lcm of two naturals is divisible from the left.
theorem lcm_divides_left(a: Nat, b: Nat) {
    a.divides(a.lcm(b))
}

/// The lcm of two naturals is divisible from the right.
theorem lcm_divides_right(a: Nat, b: Nat) {
    b.divides(a.lcm(b))
}

/// The least common multiple is symmetric in its arguments.
theorem lcm_comm(a: Nat, b: Nat) {
    a.lcm(b) = b.lcm(a)
}

/// One on the left is the identity for lcm.
theorem lcm_one_left(a: Nat) {
    Nat.1.lcm(a) = a
}

/// One on the right is the identity for lcm.
theorem lcm_one_right(a: Nat) {
    a.lcm(Nat.1) = a
}

/// Universal property: any common multiple of a and b is divisible by lcm(a, b).
theorem lcm_divides_of_common(a: Nat, b: Nat, c: Nat) {
    a.divides(c) and b.divides(c) implies a.lcm(b).divides(c)
}

/// The least common multiple is associative.
theorem lcm_assoc(a: Nat, b: Nat, c: Nat) {
    (a.lcm(b)).lcm(c) = a.lcm(b.lcm(c))
}

// congruence.ac
from nat import mod_mod, add_mod

attributes Nat {
    /// True if this number is congruent to b modulo n. Defined as the equality
    /// of the two values' remainders when divided by n. This avoids the
    /// truncating subtraction on Nat that the divisibility-based form would
    /// require.
    define congr_mod(self, b: Nat, n: Nat) -> Bool {
        self.mod(n) = b.mod(n)
    }
}

/// Congruence modulo n is reflexive.
theorem congr_mod_refl(a: Nat, n: Nat) {
    a.congr_mod(a, n)
}

/// Congruence modulo n is symmetric.
theorem congr_mod_symm(a: Nat, b: Nat, n: Nat) {
    a.congr_mod(b, n) implies b.congr_mod(a, n)
}

/// Congruence modulo n is transitive.
theorem congr_mod_trans(a: Nat, b: Nat, c: Nat, n: Nat) {
    a.congr_mod(b, n) and b.congr_mod(c, n) implies a.congr_mod(c, n)
}

/// A natural is congruent to its remainder modulo n.
theorem mod_congr_mod_self(a: Nat, n: Nat) {
    a.mod(n).congr_mod(a, n)
}

/// The mod result is strictly less than the modulus when the modulus is nonzero.
theorem mod_lt(a: Nat, n: Nat) {
    n != Nat.0 implies a.mod(n) < n
}

/// Congruence modulo n is preserved by addition.
theorem congr_mod_add(a: Nat, b: Nat, c: Nat, d: Nat, n: Nat) {
    a.congr_mod(c, n) and b.congr_mod(d, n) implies (a + b).congr_mod(c + d, n)
}

/// Congruence modulo n is preserved by multiplication.
theorem congr_mod_mul(a: Nat, b: Nat, c: Nat, d: Nat, n: Nat) {
    a.congr_mod(c, n) and b.congr_mod(d, n) implies (a * b).congr_mod(c * d, n)
}

/// Congruence modulo n is preserved by raising to a natural-number power.
theorem congr_mod_pow(a: Nat, b: Nat, n: Nat, k: Nat) {
    a.congr_mod(b, n) implies a.pow(k).congr_mod(b.pow(k), n)
}


// congr_int.ac
from nat import mod_by_zero
from int import Int, abs, abs_from_nat, add_from_nat, mul_nat_from_nat_left, mul_nat_from_nat_right,
    div_imp_div_abs
from zmod import int_mod_rel

numerals Int

/// Bridge helper: the embedding of a product factors as a product of embeddings.
theorem mul_from_nat(a: Nat, b: Nat) {
    Int.from_nat(a) * Int.from_nat(b) = Int.from_nat(a * b)
}


// bezout.ac
/// Bezout's identity on the natural numbers: the gcd of two naturals is an
/// integer-linear combination of those naturals.
theorem nat_bezout(a: Nat, b: Nat) {
    exists(x: Int, y: Int) {
        x * Int.from_nat(a) + y * Int.from_nat(b) = Int.from_nat(a.gcd(b))
    }
}

// crt.ac
from int import div_from_nat, div_trans, div_abs_imp_div
from data.int.int_residue import int_has_nat_residue
from zmod import int_mod_rel_is_equivalence
from data.basic.relation_basic import is_transitive

/// CRT combine on the naturals: when m and n are coprime and a is congruent to
/// b modulo each of them, a is congruent to b modulo their product.
theorem nat_congr_combine_coprime(m: Nat, n: Nat, a: Nat, b: Nat) {
    m.coprime(n) and a.congr_mod(b, m) and a.congr_mod(b, n)
        implies a.congr_mod(b, m * n)
}


// pairwise_coprime.ac
/// True if every element of the tail is coprime with the given head.
define coprime_with_all(head: Nat, tail: List[Nat]) -> Bool {
    match tail {
        List.nil {
            true
        }
        List.cons(t_head, t_tail) {
            head.coprime(t_head) and coprime_with_all(head, t_tail)
        }
    }
}

/// True if every pair of distinct elements in the list is coprime.
define pairwise_coprime(list: List[Nat]) -> Bool {
    match list {
        List.nil {
            true
        }
        List.cons(head, tail) {
            coprime_with_all(head, tail) and pairwise_coprime(tail)
        }
    }
}


// crt_list.ac
from pair import Pair

/// True if c satisfies every congruence requirement in a list of
/// (modulus, residue) pairs, i.e., for each (m, r) pair, c is congruent to
/// r modulo m.
define satisfies_all(c: Nat, system: List[Pair[Nat, Nat]]) -> Bool {
    match system {
        List.nil {
            true
        }
        List.cons(head, tail) {
            c.congr_mod(head.second, head.first) and satisfies_all(c, tail)
        }
    }
}

/// Project a system of (modulus, residue) pairs to its list of moduli.
define system_moduli(system: List[Pair[Nat, Nat]]) -> List[Nat] {
    match system {
        List.nil {
            List.nil[Nat]
        }
        List.cons(head, tail) {
            List.cons(head.first, system_moduli(tail))
        }
    }
}

/// The combined modulus for a system: the product of all moduli.
define system_modulus(system: List[Pair[Nat, Nat]]) -> Nat {
    product[Nat](system_moduli(system))
}

/// True if every modulus in the system is positive (nonzero).
define every_modulus_positive(system: List[Pair[Nat, Nat]]) -> Bool {
    match system {
        List.nil {
            true
        }
        List.cons(head, tail) {
            head.first != Nat.0 and every_modulus_positive(tail)
        }
    }
}

/// Any two solutions of a pairwise-coprime positive list system are congruent
/// modulo the combined system modulus.
theorem satisfies_all_unique_mod_system_modulus(system: List[Pair[Nat, Nat]], c1: Nat, c2: Nat) {
    pairwise_coprime(system_moduli(system))
        and every_modulus_positive(system)
        and satisfies_all(c1, system)
        and satisfies_all(c2, system)
        implies c1.congr_mod(c2, system_modulus(system))
}

/// A congruence modulo the system modulus is equality for representatives
/// strictly below that positive modulus.
theorem congr_mod_below_system_modulus_eq(system: List[Pair[Nat, Nat]], c1: Nat, c2: Nat) {
    every_modulus_positive(system)
        and c1 < system_modulus(system)
        and c2 < system_modulus(system)
        and c1.congr_mod(c2, system_modulus(system))
        implies c1 = c2
}

/// Any two normalized solutions of a pairwise-coprime positive list system are
/// equal.
theorem satisfies_all_unique_below_system_modulus(system: List[Pair[Nat, Nat]], c1: Nat, c2: Nat) {
    pairwise_coprime(system_moduli(system))
        and every_modulus_positive(system)
        and satisfies_all(c1, system)
        and satisfies_all(c2, system)
        and c1 < system_modulus(system)
        and c2 < system_modulus(system)
        implies c1 = c2
}

/// Every solution is congruent modulo the combined modulus to some solution
/// obtained from the list CRT existence theorem.
theorem satisfies_all_congruent_to_crt_list_solution(system: List[Pair[Nat, Nat]], c: Nat) {
    pairwise_coprime(system_moduli(system))
        and every_modulus_positive(system)
        and satisfies_all(c, system)
        implies exists(c0: Nat) {
            satisfies_all(c0, system) and c.congr_mod(c0, system_modulus(system))
        }
}


// covering_system.ac
/// True if an integer belongs to the congruence class represented by a
/// `(modulus, residue)` pair.
define congruence_class_contains(cl: Pair[Nat, Nat], x: Int) -> Bool {
    int_mod_rel(cl.first, x, Int.from_nat(cl.second))
}

/// True if an integer belongs to at least one congruence class in a list.
define covers_int(system: List[Pair[Nat, Nat]], x: Int) -> Bool {
    match system {
        List.nil {
            false
        }
        List.cons(head, tail) {
            congruence_class_contains(head, x) or covers_int(tail, x)
        }
    }
}

/// The empty list covers no integer.
theorem covers_int_nil_false(x: Int) {
    not covers_int(List.nil[Pair[Nat, Nat]], x)
}

/// Membership in a cons covering list unfolds to the head class or the tail.
theorem covers_int_cons_imp(head: Pair[Nat, Nat], tail: List[Pair[Nat, Nat]], x: Int) {
    covers_int(List.cons(head, tail), x)
        implies congruence_class_contains(head, x) or covers_int(tail, x)
}

/// The head class covers every integer it contains.
theorem covers_int_cons_left(head: Pair[Nat, Nat], tail: List[Pair[Nat, Nat]], x: Int) {
    congruence_class_contains(head, x) implies covers_int(List.cons(head, tail), x)
}

/// A tail cover remains a cover after consing one more class.
theorem covers_int_cons_right(head: Pair[Nat, Nat], tail: List[Pair[Nat, Nat]], x: Int) {
    covers_int(tail, x) implies covers_int(List.cons(head, tail), x)
}

/// True if every integer is covered by the list of congruence classes.
define covers_all_int(system: List[Pair[Nat, Nat]]) -> Bool {
    forall(x: Int) {
        covers_int(system, x)
    }
}

/// A universal integer cover applies to any chosen integer.
theorem covers_all_int_apply(system: List[Pair[Nat, Nat]], x: Int) {
    covers_all_int(system) implies covers_int(system, x)
}

/// A pointwise cover of every integer is a universal integer cover.
theorem covers_all_int_intro(system: List[Pair[Nat, Nat]]) {
    (forall(x: Int) { covers_int(system, x) }) implies covers_all_int(system)
}

/// A covering system is a finite list of positive-modulus congruence classes
/// whose union is all of the integers.
define is_covering_system(system: List[Pair[Nat, Nat]]) -> Bool {
    every_modulus_positive(system) and covers_all_int(system)
}

/// A covering system has positive moduli.
theorem covering_system_moduli_positive(system: List[Pair[Nat, Nat]]) {
    is_covering_system(system) implies every_modulus_positive(system)
}

/// A covering system covers every integer.
theorem covering_system_covers_int(system: List[Pair[Nat, Nat]], x: Int) {
    is_covering_system(system) implies covers_int(system, x)
}

/// Adding one positive congruence class to a covering system preserves coverage.
theorem covering_system_cons_of_tail(head: Pair[Nat, Nat], tail: List[Pair[Nat, Nat]]) {
    head.first != Nat.0 and is_covering_system(tail)
        implies is_covering_system(List.cons(head, tail))
}

/// Congruence modulo one is the universal integer relation.
theorem int_mod_rel_one(x: Int, y: Int) {
    int_mod_rel(Nat.1, x, y)
}

/// Integer congruence is symmetric.
theorem int_mod_rel_symm(m: Nat, x: Int, y: Int) {
    int_mod_rel(m, x, y) implies int_mod_rel(m, y, x)
}

/// Integer congruence is transitive.
theorem int_mod_rel_trans(m: Nat, x: Int, y: Int, z: Int) {
    int_mod_rel(m, x, y) and int_mod_rel(m, y, z) implies int_mod_rel(m, x, z)
}

/// A congruence class is closed under congruence modulo its modulus.
theorem congruence_class_contains_of_congruent(cl: Pair[Nat, Nat], x: Int, y: Int) {
    int_mod_rel(cl.first, x, y) and congruence_class_contains(cl, y)
        implies congruence_class_contains(cl, x)
}

/// The congruence class modulo one contains every integer.
theorem congruence_class_mod_one_contains(r: Nat, x: Int) {
    congruence_class_contains(Pair.new(Nat.1, r), x)
}

/// A single congruence class modulo one is a covering system.
theorem singleton_mod_one_covering_system(r: Nat) {
    is_covering_system(List.cons(Pair.new(Nat.1, r), List.nil[Pair[Nat, Nat]]))
}

/// The head modulus divides the combined modulus of a cons system.
theorem head_modulus_divides_cons_modulus(head: Pair[Nat, Nat], tail: List[Pair[Nat, Nat]]) {
    head.first.divides(system_modulus(List.cons(head, tail)))
}

/// The tail combined modulus divides the combined modulus of a cons system.
theorem tail_modulus_divides_cons_modulus(head: Pair[Nat, Nat], tail: List[Pair[Nat, Nat]]) {
    system_modulus(tail).divides(system_modulus(List.cons(head, tail)))
}

/// True if every residue below the combined system modulus is covered.
define covers_all_residues(system: List[Pair[Nat, Nat]]) -> Bool {
    forall(r: Nat) {
        r < system_modulus(system) implies covers_int(system, Int.from_nat(r))
    }
}

/// Pointwise residue coverage gives the finite residue-check predicate.
theorem covers_all_residues_intro(system: List[Pair[Nat, Nat]]) {
    (forall(r: Nat) {
        r < system_modulus(system) implies covers_int(system, Int.from_nat(r))
    }) implies covers_all_residues(system)
}

/// The finite residue-check predicate applies to each residue below the
/// combined modulus.
theorem covers_all_residues_apply(system: List[Pair[Nat, Nat]], r: Nat) {
    covers_all_residues(system) and r < system_modulus(system)
        implies covers_int(system, Int.from_nat(r))
}

/// Every covering system covers all residues below its combined modulus.
theorem covering_system_covers_all_residues(system: List[Pair[Nat, Nat]]) {
    is_covering_system(system) implies covers_all_residues(system)
}


// modular_inverse.ac
from zmod import int_mod_rel_mul_compatible
from algebra.ring.ring import mul_neg_left

attributes Nat {
    /// True when b is a modular inverse of self modulo n: their product is
    /// congruent to one modulo n.
    define is_mod_inv(self, b: Nat, n: Nat) -> Bool {
        (self * b).congr_mod(Nat.1, n)
    }
}

/// The modular inverse of a modulo n: a witnessing Nat with `a * mod_inv(a, n)`
/// congruent to one modulo n when a is coprime to n. When the inverse does not
/// exist (a not coprime to n), the value is the placeholder zero.
let mod_inv(a: Nat, n: Nat) -> b: Nat satisfy {
    if a.coprime(n) {
        a.is_mod_inv(b, n)
    } else {
        b = Nat.0
    }
}

/// Modular cancellation: if `a` is coprime to `n`, then `a * x ≡ a * y (mod n)`
/// implies `x ≡ y (mod n)`. The standard proof multiplies both sides by an
/// inverse of `a` modulo `n` and collapses the resulting `1 * x` and `1 * y`.
theorem cancel_coprime(a: Nat, n: Nat, x: Nat, y: Nat) {
    a.coprime(n) and (a * x).congr_mod(a * y, n)
        implies x.congr_mod(y, n)
}


// fermat.ac
from combinatorics import binomial_term, binomial, choose_zero, choose_n
from nat import factorial_step, divides_lte, divides_add, div_imp_mod, suc_sub_one, add_imp_sub
from nat import exp_one, exp_add, zero_exp, one_exp
from list import partial, partial_split_first_last, partial_split_last, partial_zero
from data.basic.functions import compose

/// Inductive predicate used in prime_does_not_divide_factorial: at each Nat x,
/// p does not divide x.factorial whenever x < p.
define p_no_div_fact_pred(p: Nat) -> (Nat -> Bool) {
    function(x: Nat) {
        x < p implies not p.divides(x.factorial)
    }
}

/// Divisibility predicate for `divides_partial_nat`: at each Nat m, if d
/// divides every f(k) for k < m, then d divides partial(f, m).
define divides_partial_pred(d: Nat, f: Nat -> Nat) -> (Nat -> Bool) {
    function(m: Nat) {
        (forall(k: Nat) { k < m implies d.divides(f(k)) })
            implies d.divides(partial[Nat](f, m))
    }
}

/// Inductive predicate for Fermat's little theorem: a.pow(p) is congruent to a
/// modulo p.
define fermat_pred(p: Nat) -> (Nat -> Bool) {
    function(a: Nat) {
        a.pow(p).congr_mod(a, p)
    }
}

/// Inductive predicate for rsa_pow_congr.
define rsa_pow_pred(m: Nat, p: Nat) -> (Nat -> Bool) {
    function(k: Nat) {
        m.pow(k * (p - Nat.1) + Nat.1).congr_mod(m, p)
    }
}

/// RSA power congruence: for prime p, every exponent of the form k * (p - 1) + 1
/// behaves like exponent 1 modulo p.
theorem rsa_pow_congr(p: Nat, m: Nat, k: Nat) {
    p.is_prime implies m.pow(k * (p - Nat.1) + Nat.1).congr_mod(m, p)
}


// totient.ac
from nat import small_mod
from list import list_contains_implies_count_geq_one, unique_implies_no_duplicate
from list import map_length, map_contains, map_contains_of_contains

/// The number of natural numbers in `[0, k)` that are coprime to `n`. Used to
/// build Euler's totient via `nat_totient(n) = count_coprime_to(n, n)`.
define count_coprime_to(n: Nat, k: Nat) -> Nat {
    match k {
        Nat.zero {
            Nat.0
        }
        Nat.suc(j) {
            if j.coprime(n) {
                count_coprime_to(n, j) + Nat.1
            } else {
                count_coprime_to(n, j)
            }
        }
    }
}

attributes Nat {
    /// Euler's totient: the number of natural numbers in `[0, n)` that are
    /// coprime to `n`. Reduces to `n - 1` for prime `n` and to
    /// `(p - 1) * (q - 1)` for distinct primes `p`, `q`.
    define totient(self) -> Nat {
        count_coprime_to(self, self)
    }
}

/// Euler's totient as an arithmetic function.
let nat_totient: Nat -> Nat = function(n: Nat) { n.totient }

/// Euler's totient vanishes at zero.
theorem nat_totient_zero {
    nat_totient(Nat.0) = Nat.0
}

/// Euler's totient is one at one.
theorem nat_totient_one {
    nat_totient(Nat.1) = Nat.1
}

/// Inductive predicate for `totient_prime`: `count_coprime_to(p, k)` equals
/// `k - 1` for `1 <= k <= p` when `p` is prime.
define totient_prime_pred(p: Nat) -> (Nat -> Bool) {
    function(k: Nat) {
        Nat.1 <= k and k <= p implies count_coprime_to(p, k) = k - Nat.1
    }
}

/// The number of natural numbers in `[0, k)` divisible by `d`.
/// Used for inclusion-exclusion-style counts that feed into totient identities.
define count_multiples(d: Nat, k: Nat) -> Nat {
    match k {
        Nat.zero {
            Nat.0
        }
        Nat.suc(j) {
            if d.divides(j) {
                count_multiples(d, j) + Nat.1
            } else {
                count_multiples(d, j)
            }
        }
    }
}

/// Inductive predicate for `count_multiples_block_aligned`. It tracks the
/// offset `j` from the starting multiple `k`. For any `j` in `[0, d]`, the
/// count of multiples of `d` in `[0, k + j)` exceeds the count in `[0, k)`
/// by `0` if `j = 0` and by `1` otherwise.
define block_aligned_pred(d: Nat, k: Nat) -> (Nat -> Bool) {
    function(j: Nat) {
        j <= d and Nat.0 < j
            implies count_multiples(d, k + j) = count_multiples(d, k) + Nat.1
    }
}

/// Predicate version of `count_multiples_div`.
define count_multiples_div_pred(d: Nat) -> (Nat -> Bool) {
    function(q: Nat) {
        d != Nat.0 implies count_multiples(d, d * q) = q
    }
}

/// Counterpart to `count_coprime_to`: the number of natural numbers in
/// `[0, k)` that are NOT coprime to `n`. Together with `count_coprime_to` it
/// partitions `[0, k)`.
define count_not_coprime_to(n: Nat, k: Nat) -> Nat {
    match k {
        Nat.zero {
            Nat.0
        }
        Nat.suc(j) {
            if j.coprime(n) {
                count_not_coprime_to(n, j)
            } else {
                count_not_coprime_to(n, j) + Nat.1
            }
        }
    }
}

/// Inductive predicate for the coprime/non-coprime conservation law.
define coprime_partition_pred(n: Nat) -> (Nat -> Bool) {
    function(k: Nat) {
        count_coprime_to(n, k) + count_not_coprime_to(n, k) = k
    }
}

/// Inclusion-exclusion identity over `[0, k)`: for distinct primes `p`, `q`,
/// the not-coprime-to-`p*q` count combined with the multiples-of-`p*q` count
/// equals the sum of multiples of `p` and multiples of `q`.
define ie_pq_pred(p: Nat, q: Nat) -> (Nat -> Bool) {
    function(k: Nat) {
        count_not_coprime_to(p * q, k) + count_multiples(p * q, k) =
            count_multiples(p, k) + count_multiples(q, k)
    }
}

/// Inductive predicate for the prime-square count equivalence.
define pp_count_pred(p: Nat) -> (Nat -> Bool) {
    function(k: Nat) {
        count_not_coprime_to(p * p, k) = count_multiples(p, k)
    }
}

/// Inductive predicate for `coprime_pow_iff`: at each `n`, coprimality with
/// `p^(n.suc)` agrees with coprimality with `p`.
define coprime_pow_pred(k: Nat, p: Nat) -> (Nat -> Bool) {
    function(n: Nat) {
        k.coprime(p.pow(n.suc)) = k.coprime(p)
    }
}

/// Inductive predicate for the prime-power count equivalence.
define ppow_count_pred(p: Nat, n: Nat) -> (Nat -> Bool) {
    function(k: Nat) {
        count_not_coprime_to(p.pow(n.suc), k) = count_multiples(p, k)
    }
}

/// The list of natural numbers in `[0, k)` that are coprime to `n`, in
/// descending order. Mirror of `count_coprime_to` shaped to allow product /
/// permutation arguments downstream (e.g. for general Euler).
define coprime_residues_below(n: Nat, k: Nat) -> List[Nat] {
    match k {
        Nat.zero {
            List.nil[Nat]
        }
        Nat.suc(j) {
            if j.coprime(n) {
                List.cons(j, coprime_residues_below(n, j))
            } else {
                coprime_residues_below(n, j)
            }
        }
    }
}

/// The full coprime-residue list for modulus `n`: integers in `[0, n)`
/// coprime to `n`. Has length `nat_totient(n)`.
define coprime_residues(n: Nat) -> List[Nat] {
    coprime_residues_below(n, n)
}

/// Inductive predicate for `coprime_residues_below_length`.
define coprime_residues_length_pred(n: Nat) -> (Nat -> Bool) {
    function(k: Nat) {
        coprime_residues_below(n, k).length = count_coprime_to(n, k)
    }
}

/// `coprime_residues(n).length = nat_totient(n)`.
theorem coprime_residues_length(n: Nat) {
    coprime_residues(n).length = n.totient
}

/// Euler's totient is multiplicative on positive coprime arguments.
theorem totient_mul_coprime_positive(m: Nat, n: Nat) {
    Nat.0 < m and Nat.0 < n and m.coprime(n) implies
        (m * n).totient = m.totient * n.totient
}

/// Euler's totient is multiplicative on one pair of coprime arguments.
theorem nat_totient_mul_coprime(a: Nat, b: Nat) {
    a.coprime(b) implies nat_totient(a * b) = nat_totient(a) * nat_totient(b)
}

/// Inductive predicate for `coprime_residues_below_all_coprime`.
define cr_all_coprime_pred(n: Nat) -> (Nat -> Bool) {
    function(k: Nat) {
        forall(x: Nat) {
            coprime_residues_below(n, k).contains(x) implies x.coprime(n)
        }
    }
}

/// Inductive predicate for `coprime_residues_below_all_below`.
define cr_all_below_pred(n: Nat) -> (Nat -> Bool) {
    function(k: Nat) {
        forall(x: Nat) {
            coprime_residues_below(n, k).contains(x) implies x < k
        }
    }
}

/// Inductive predicate for `coprime_residues_below_contains`: every coprime
/// `x < k` lies in the residue list.
define cr_contains_pred(n: Nat) -> (Nat -> Bool) {
    function(k: Nat) {
        forall(x: Nat) {
            x < k and x.coprime(n) implies coprime_residues_below(n, k).contains(x)
        }
    }
}

/// Inductive predicate for `coprime_residues_below_unique`.
define cr_unique_pred(n: Nat) -> (Nat -> Bool) {
    function(k: Nat) {
        coprime_residues_below(n, k).is_unique
    }
}

/// Top-level point function `x -> (a * x).mod(n)` so that `map` clients
/// don't have to inline the lambda. Using a named function rather than an
/// anonymous one inside `define mul_mod_residues` keeps the equational
/// `mul_mod_residues(n, a) = map(coprime_residues(n), mul_mod_fn(n, a))`
/// reflexive under `acorn check`'s certificate replay.
define mul_mod_fn(n: Nat, a: Nat) -> (Nat -> Nat) {
    function(x: Nat) { (a * x).mod(n) }
}

/// The map `x -> (a * x).mod(n)` applied to every element of
/// `coprime_residues(n)`. By multiplication-by-unit membership and
/// surjectivity, this list has the same elements as `coprime_residues(n)`.
define mul_mod_residues(n: Nat, a: Nat) -> List[Nat] {
    map(coprime_residues(n), mul_mod_fn(n, a))
}

/// Forward inclusion: every element of `mul_mod_residues(n, a)` lies in
/// `coprime_residues(n)` (assuming `n > 0` and `a` coprime to `n`).
theorem mul_mod_residues_in_coprime(n: Nat, a: Nat, y: Nat) {
    n != Nat.0 and a.coprime(n) and mul_mod_residues(n, a).contains(y)
        implies coprime_residues(n).contains(y)
}

/// Backward inclusion: every element of `coprime_residues(n)` is in
/// `mul_mod_residues(n, a)` (assuming `n > 0` and `a` coprime to `n`).
theorem coprime_in_mul_mod_residues(n: Nat, a: Nat, y: Nat) {
    n != Nat.0 and a.coprime(n) and coprime_residues(n).contains(y)
        implies mul_mod_residues(n, a).contains(y)
}

/// Inductive predicate for `mul_mod_residues_below_unique`: for `n != 0` and
/// `a` coprime to `n`, the image of `coprime_residues_below(n, k)` under
/// `mul_mod_fn(n, a)` is `is_unique`, provided `k <= n`.
define mul_mod_residues_below_unique_pred(n: Nat, a: Nat) -> (Nat -> Bool) {
    function(k: Nat) {
        k <= n implies
            map(coprime_residues_below(n, k), mul_mod_fn(n, a)).is_unique
    }
}

/// `is_permutation(mul_mod_residues(n, a), coprime_residues(n))` for `n > 0`
/// and `a` coprime to `n`. Combines the bidirectional membership with
/// uniqueness on both sides: each element contributes count `0` or `1`, and
/// the contains-statuses agree.
theorem mul_mod_residues_is_permutation(n: Nat, a: Nat) {
    n != Nat.0 and a.coprime(n)
        implies is_permutation(mul_mod_residues(n, a), coprime_residues(n))
}

/// Top-level scalar-multiplication function for use with `map`. Like
/// `mul_mod_fn`, naming this avoids inline-lambda issues in `acorn check`'s
/// certificate replay.
define scalar_mul_fn(a: Nat) -> (Nat -> Nat) {
    function(x: Nat) { a * x }
}

/// Inductive predicate for `product_map_scalar`.
define product_map_scalar_pred(a: Nat) -> (List[Nat] -> Bool) {
    function(l: List[Nat]) {
        product[Nat](map(l, scalar_mul_fn(a))) = a.pow(l.length) * product[Nat](l)
    }
}

/// `product(map(L, λx. a*x)) = a^|L| * product(L)` for Nat lists.
/// Pulls the constant factor `a` out of every term in the list product,
/// raised to the list length. Building block for Euler-style arguments.
theorem product_map_scalar(a: Nat, l: List[Nat]) {
    product[Nat](map(l, scalar_mul_fn(a))) = a.pow(l.length) * product[Nat](l)
}

/// Inductive predicate for `product_coprime_of_all`.
define product_coprime_pred(n: Nat) -> (List[Nat] -> Bool) {
    function(l: List[Nat]) {
        (forall(x: Nat) { l.contains(x) implies x.coprime(n) })
            implies product[Nat](l).coprime(n)
    }
}

/// `product(coprime_residues(n)).coprime(n)`. Specialisation of
/// `product_coprime_of_all` to the units list — every element is coprime to
/// `n` by construction (`coprime_residues_all_coprime`), so their product is
/// too.
theorem product_coprime_residues_coprime(n: Nat) {
    product[Nat](coprime_residues(n)).coprime(n)
}

/// Helper: rewrite `product(map(cons(h, t), f))` as `f(h) * product(map(t, f))`.
theorem product_map_cons(f: Nat -> Nat, head: Nat, tail: List[Nat]) {
    product[Nat](map(List.cons(head, tail), f)) =
        f(head) * product[Nat](map(tail, f))
}

/// Inductive predicate for `product_mul_mod_congr_scalar`: at each list `l`,
/// the product of `map(l, mul_mod_fn(n, a))` is congruent modulo `n` to the
/// product of `map(l, scalar_mul_fn(a))`.
define pmmcs_pred(n: Nat, a: Nat) -> (List[Nat] -> Bool) {
    function(l: List[Nat]) {
        product[Nat](map(l, mul_mod_fn(n, a))).congr_mod(
            product[Nat](map(l, scalar_mul_fn(a))), n)
    }
}

/// Inductive predicate for `product_map_mul_mod`: replacing each list entry
/// `x` by `(a * x).mod(n)` only changes the list product by a power of `a`,
/// modulo `n`.
define product_map_mul_mod_pred(n: Nat, a: Nat) -> (List[Nat] -> Bool) {
    function(l: List[Nat]) {
        product[Nat](map(l, mul_mod_fn(n, a))).congr_mod(
            a.pow(l.length) * product[Nat](l), n)
    }
}


// multiplicative_order.ac
from nat import has_min, is_min, false_below
from nat import exp_mul
from nat import division_theorem, lte_imp_not_lt, add_sub, lt_add_left, lt_and_lte, trichotomy

/// The positive exponents at which `a` is congruent to `1` modulo `n`.
define multiplicative_order_witness(a: Nat, n: Nat) -> (Nat -> Bool) {
    function(k: Nat) {
        Nat.0 < k and a.pow(k).congr_mod(Nat.1, n)
    }
}

/// True when `k` is the least positive exponent with `a^k ≡ 1 (mod n)`.
define is_multiplicative_order_mod(a: Nat, n: Nat, k: Nat) -> Bool {
    is_min(multiplicative_order_witness(a, n), k)
}

/// The multiplicative order of `a` modulo `n`, with placeholder value `0`
/// outside the positive-coprime domain.
let multiplicative_order_mod(a: Nat, n: Nat) -> k: Nat satisfy {
    (n != Nat.0 and a.coprime(n) and is_multiplicative_order_mod(a, n, k))
        or ((n = Nat.0 or not a.coprime(n)) and k = Nat.0)
}


// quadratic_residue.ac
from nat import sq_eq_mul
from nat import div_sub_mod, mod_of_zero, sub_zero

/// True when `a` is a square modulo `n`.
define is_quadratic_residue_mod(a: Nat, n: Nat) -> Bool {
    exists(x: Nat) { x.pow(Nat.2).congr_mod(a, n) }
}

/// True when `a` is represented by the square of a unit modulo `n`.
define is_unit_quadratic_residue_mod(a: Nat, n: Nat) -> Bool {
    exists(x: Nat) { x.coprime(n) and x.pow(Nat.2).congr_mod(a, n) }
}


// primitive_root.ac
from nat import divides_cancel_right

/// True when `a` is represented by a power of `g` modulo `n`.
define is_power_of_mod(a: Nat, g: Nat, n: Nat) -> Bool {
    exists(k: Nat) { a.congr_mod(g.pow(k), n) }
}

/// The powers of `g` cover every reduced residue class modulo `n`.
define powers_cover_units_mod(g: Nat, n: Nat) -> Bool {
    forall(a: Nat) { a.coprime(n) implies is_power_of_mod(a, g, n) }
}

/// An order-`2*h` generator interface with explicit unit coverage.
define is_order_double_unit_generator_mod(g: Nat, n: Nat, h: Nat) -> Bool {
    n != Nat.0 and g.coprime(n) and h != Nat.0 and
    is_multiplicative_order_mod(g, n, Nat.2 * h) and powers_cover_units_mod(g, n)
}


// wilson.ac
from list import unique_implies_tail_unique
from list import remove_one_contains_other, remove_one_unique, remove_one_unique_not_contains_self,
    unique_same_contains_imp_permutation
from nat import divides_factorial, divides_sub, divisor_lt, sub_one_lt

/// The induction predicate used to identify the product of prime reduced
/// residues below a bound with the preceding factorial.
define prime_coprime_residues_product_pred(p: Nat) -> (Nat -> Bool) {
    function(k: Nat) {
        k <= p implies
            product[Nat](coprime_residues_below(p, k)) = (k - Nat.1).factorial
    }
}

/// The modular inverse reduced into the canonical residue range.
define inv_mod_fn(n: Nat) -> (Nat -> Nat) {
    function(x: Nat) { mod_inv(x, n).mod(n) }
}

/// The reduced residues mapped through normalized modular inverse.
define inv_mod_residues(n: Nat) -> List[Nat] {
    map(coprime_residues(n), inv_mod_fn(n))
}

/// Multiplies a value by its normalized modular inverse.
define mul_inv_mod_fn(n: Nat) -> (Nat -> Nat) {
    function(x: Nat) { x * inv_mod_fn(n)(x) }
}

/// The reduced-residue list with the two fixed residues removed.
define wilson_nonfixed_residues(p: Nat) -> List[Nat] {
    coprime_residues(p).remove_one(Nat.1).remove_one(p - Nat.1)
}

/// True if every element of a list is a reduced residue modulo `p`.
define wilson_items_are_residues(p: Nat, items: List[Nat]) -> Bool {
    forall(x: Nat) { items.contains(x) implies coprime_residues(p).contains(x) }
}

/// True if a list excludes the two fixed residues `1` and `p - 1`.
define wilson_items_nonfixed(p: Nat, items: List[Nat]) -> Bool {
    forall(x: Nat) { items.contains(x) implies x != Nat.1 and x != p - Nat.1 }
}

/// True if a list is closed under normalized modular inversion.
define wilson_items_inv_closed(p: Nat, items: List[Nat]) -> Bool {
    forall(x: Nat) { items.contains(x) implies items.contains(inv_mod_fn(p)(x)) }
}

/// True if a list of residues can be paired by normalized modular inversion,
/// with the two fixed residues `1` and `p - 1` removed.
define wilson_pairable_list(p: Nat, items: List[Nat]) -> Bool {
    items.is_unique and wilson_items_are_residues(p, items) and
    wilson_items_nonfixed(p, items) and wilson_items_inv_closed(p, items)
}

/// Inductive predicate for uniqueness of normalized inverse images below a
/// bound.
define inv_mod_residues_below_unique_pred(n: Nat) -> (Nat -> Bool) {
    function(k: Nat) {
        k <= n implies map(coprime_residues_below(n, k), inv_mod_fn(n)).is_unique
    }
}

/// Predicate for products of `x * inv(x)` over a list.
define product_mul_inv_congr_one_pred(n: Nat) -> (List[Nat] -> Bool) {
    function(items: List[Nat]) {
        (forall(x: Nat) { items.contains(x) implies x.coprime(n) }) implies
            product[Nat](map(items, mul_inv_mod_fn(n))).congr_mod(Nat.1, n)
    }
}

/// Predicate for distributing the product of pointwise `x * inv(x)`.
define product_mul_inv_distrib_pred(n: Nat) -> (List[Nat] -> Bool) {
    function(items: List[Nat]) {
        product[Nat](map(items, mul_inv_mod_fn(n))) =
            product[Nat](items) * product[Nat](map(items, inv_mod_fn(n)))
    }
}

/// Bounded induction predicate for pairable-list products.
define wilson_pairable_product_bound_pred(p: Nat) -> (Nat -> Bool) {
    function(m: Nat) {
        forall(items: List[Nat]) {
            items.length <= m and p.is_prime and wilson_pairable_list(p, items)
                implies product[Nat](items).congr_mod(Nat.1, p)
        }
    }
}

/// Wilson's factorial congruence holds for prime moduli.
theorem prime_imp_wilson_factorial_congr(p: Nat) {
    p.is_prime implies (p - Nat.1).factorial.congr_mod(p - Nat.1, p)
}

/// Wilson's congruence implies primality. This is the easier half of Wilson's
/// theorem; the prime-to-congruence direction still needs the inverse-pairing
/// product argument.
theorem wilson_factorial_congr_imp_prime(p: Nat) {
    p != Nat.1 and (p - Nat.1).factorial.congr_mod(p - Nat.1, p)
        implies p.is_prime
}


// arithmetic_functions.ac
from nat import add_to_zero
from nat import pow_distrib_mul

/// The constant arithmetic function with value one.
let nat_one_arithmetic_fn: Nat -> Nat = function(n: Nat) { Nat.1 }

/// The identity arithmetic function.
let nat_identity_arithmetic_fn: Nat -> Nat = function(n: Nat) { n }

/// The power arithmetic function `n -> n^k`.
define nat_power_arithmetic_fn(k: Nat) -> (Nat -> Nat) {
    function(n: Nat) { n.pow(k) }
}

/// The pointwise product of two arithmetic functions.
define arithmetic_fn_mul(f: Nat -> Nat, g: Nat -> Nat) -> (Nat -> Nat) {
    function(n: Nat) { f(n) * g(n) }
}

/// True if an arithmetic function is multiplicative on coprime arguments.
define is_multiplicative_nat_fn(f: Nat -> Nat) -> Bool {
    f(Nat.1) = Nat.1 and forall(a: Nat, b: Nat) {
        a.coprime(b) implies f(a * b) = f(a) * f(b)
    }
}

/// True if an arithmetic function is multiplicative on all arguments.
define is_completely_multiplicative_nat_fn(f: Nat -> Nat) -> Bool {
    f(Nat.1) = Nat.1 and forall(a: Nat, b: Nat) {
        f(a * b) = f(a) * f(b)
    }
}

/// The constant arithmetic function with value zero.
let nat_zero_arithmetic_fn: Nat -> Nat = function(n: Nat) { Nat.0 }

/// The pointwise sum of two arithmetic functions.
define arithmetic_fn_add(f: Nat -> Nat, g: Nat -> Nat) -> (Nat -> Nat) {
    function(n: Nat) { f(n) + g(n) }
}

/// The Dirichlet identity arithmetic function: 1 at n = 1 and 0 elsewhere.
define nat_dirichlet_unit_fn(n: Nat) -> Nat {
    if n = Nat.1 { Nat.1 } else { Nat.0 }
}

/// True if `f(n)` is positive for every positive `n`. A handy hypothesis when
/// arguing about multiplicativity over coprime products of positive arguments.
define positive_on_positive(f: Nat -> Nat) -> Bool {
    forall(n: Nat) { Nat.0 < n implies Nat.0 < f(n) }
}


// divisor_sum.ac
from list import map_sum_add
from algebra.add_semigroup import add_fn

/// The list of divisors `d` of `n` with `0 < d <= k`, in descending order.
/// A helper that lets the divisor list be built by induction on the bound.
define divisors_up_to(n: Nat, k: Nat) -> List[Nat] {
    match k {
        Nat.zero {
            List.nil[Nat]
        }
        Nat.suc(j) {
            if k.divides(n) {
                List.cons(k, divisors_up_to(n, j))
            } else {
                divisors_up_to(n, j)
            }
        }
    }
}

/// The list of positive divisors of `n` up to `n` itself, in descending order.
/// For `n >= 1` this enumerates every positive divisor of `n`.
define divisor_list(n: Nat) -> List[Nat] {
    divisors_up_to(n, n)
}

/// The divisor-sum operator: `divisor_sum_fn(f)(n) = sum_{d | n, d <= n} f(d)`.
/// For `n >= 1` this is the usual `sum_{d | n} f(d)` from elementary number theory.
define divisor_sum_fn(f: Nat -> Nat) -> (Nat -> Nat) {
    function(n: Nat) { sum(map(divisor_list(n), f)) }
}

/// The number of positive divisors of `n` (the standard `tau` function).
define nat_tau(n: Nat) -> Nat {
    divisor_list(n).length
}

/// The sum of the positive divisors of `n` (the standard `sigma` function).
define nat_sigma(n: Nat) -> Nat {
    sum(divisor_list(n))
}

/// Inductive predicate for the prime-offset divisor list lemma.
define divisors_up_to_prime_offset_pred(p: Nat, y: Nat) -> Bool {
    Nat.1 + y < p implies
        divisors_up_to(p, Nat.1 + y) = List.cons(Nat.1, List.nil[Nat])
}

/// Inductive predicate for uniqueness of `divisors_up_to`.
define divisors_up_to_unique_pred(n: Nat, k: Nat) -> Bool {
    divisors_up_to(n, k).is_unique
}


// dirichlet.ac
from list import unique_same_contains_map_sum_eq

/// The unique cofactor `q` with `d * q = n` when `d` divides `n`, and zero
/// otherwise. Lets us write `n / d` as `divisor_quotient(n, d)` in arithmetic
/// arguments where `d` ranges over the divisors of `n`.
let divisor_quotient(n: Nat, d: Nat) -> q: Nat satisfy {
    if d.divides(n) {
        d * q = n
    } else {
        q = Nat.0
    }
}

/// The single-argument fiber of the Dirichlet convolution: at fixed `n`, this
/// is the function `d -> f(d) * g(n / d)`.
define dirichlet_term(f: Nat -> Nat, g: Nat -> Nat, n: Nat) -> (Nat -> Nat) {
    function(d: Nat) { f(d) * g(divisor_quotient(n, d)) }
}

/// The Dirichlet convolution `dirichlet_convolve(f, g)(n) = sum_{d | n} f(d) * g(n / d)`.
define dirichlet_convolve(f: Nat -> Nat, g: Nat -> Nat) -> (Nat -> Nat) {
    function(n: Nat) { sum(map(divisor_list(n), dirichlet_term(f, g, n))) }
}

/// True when the pair-product Dirichlet convolution sum may be factored.
define dirichlet_convolve_pair_product_sum_hyp(f: Nat -> Nat, g: Nat -> Nat,
    a: Nat, b: Nat) -> Bool {
    is_multiplicative_nat_fn(f) and is_multiplicative_nat_fn(g) and
    Nat.0 < a and Nat.0 < b and a.coprime(b)
}

/// Dirichlet convolution of multiplicative functions is multiplicative on
/// positive coprime arguments.
theorem dirichlet_convolve_mul_coprime_positive(f: Nat -> Nat, g: Nat -> Nat,
    a: Nat, b: Nat) {
    dirichlet_convolve_pair_product_sum_hyp(f, g, a, b) implies
        dirichlet_convolve(f, g)(a * b) =
            dirichlet_convolve(f, g)(a) * dirichlet_convolve(f, g)(b)
}

/// Dirichlet convolution preserves multiplicative arithmetic functions.
theorem dirichlet_convolve_multiplicative(f: Nat -> Nat, g: Nat -> Nat) {
    is_multiplicative_nat_fn(f) and is_multiplicative_nat_fn(g)
        implies is_multiplicative_nat_fn(dirichlet_convolve(f, g))
}

/// The number of divisors is multiplicative on positive coprime arguments.
theorem nat_tau_mul_coprime_positive(a: Nat, b: Nat) {
    Nat.0 < a and Nat.0 < b and a.coprime(b) implies
        nat_tau(a * b) = nat_tau(a) * nat_tau(b)
}

/// The number of divisors is multiplicative on coprime arguments.
theorem nat_tau_multiplicative {
    is_multiplicative_nat_fn(nat_tau)
}

/// The sum of divisors is multiplicative on positive coprime arguments.
theorem nat_sigma_mul_coprime_positive(a: Nat, b: Nat) {
    Nat.0 < a and Nat.0 < b and a.coprime(b) implies
        nat_sigma(a * b) = nat_sigma(a) * nat_sigma(b)
}

/// The sum of divisors is multiplicative on coprime arguments.
theorem nat_sigma_multiplicative {
    is_multiplicative_nat_fn(nat_sigma)
}

/// Predicate for the zero-fn induction: `sum(map(xs, h)) = 0` whenever `h` is
/// identically zero.
define sum_map_zero_pred(h: Nat -> Nat) -> (List[Nat] -> Bool) {
    function(xs: List[Nat]) {
        forall(d: Nat) { h(d) = Nat.0 } implies sum(map(xs, h)) = Nat.0
    }
}

/// Inductive predicate for the right-identity helper: at bound `k < n`, the
/// dirichlet-term sum over `divisors_up_to(n, k)` vanishes because every such
/// divisor produces a non-one cofactor for the Dirichlet unit.
define dirichlet_unit_right_below_pred(f: Nat -> Nat, n: Nat, k: Nat) -> Bool {
    k < n implies
        sum(map(divisors_up_to(n, k),
            dirichlet_term(f, nat_dirichlet_unit_fn, n))) = Nat.0
}

/// Inductive predicate for the left-identity helper: at any bound `k >= 1`,
/// the dirichlet-term sum with the Dirichlet unit on the left equals `f(n)`,
/// because only the divisor `d = 1` contributes.
define dirichlet_unit_left_below_pred(f: Nat -> Nat, n: Nat, k: Nat) -> Bool {
    Nat.1 <= k implies
        sum(map(divisors_up_to(n, k),
            dirichlet_term(nat_dirichlet_unit_fn, f, n))) = f(n)
}

/// The cofactor-indexed Dirichlet term: at index `d`, the swapped-pair
/// Dirichlet term evaluated at the cofactor `n / d`.
define dirichlet_cofactor_term(f: Nat -> Nat, g: Nat -> Nat, n: Nat) -> (Nat -> Nat) {
    function(d: Nat) { dirichlet_term(g, f, n)(divisor_quotient(n, d)) }
}

/// Inductive predicate for the swap form: at any bound `k`, the
/// dirichlet-term sum over `divisors_up_to(n, k)` equals the
/// cofactor-indexed swapped-pair sum.
define dirichlet_swap_below_pred(f: Nat -> Nat, g: Nat -> Nat, n: Nat, k: Nat) -> Bool {
    Nat.0 < n implies
        sum(map(divisors_up_to(n, k), dirichlet_term(f, g, n))) =
        sum(map(divisors_up_to(n, k), dirichlet_cofactor_term(f, g, n)))
}

/// The cofactor map at fixed `n`, viewed as a unary function from divisors of
/// `n` to divisors of `n`.
define nat_divisor_quotient_fn(n: Nat) -> (Nat -> Nat) {
    function(d: Nat) { divisor_quotient(n, d) }
}

/// The list-image of the divisors of `n` under the cofactor map.
define cofactor_image_list(n: Nat) -> List[Nat] {
    map(divisor_list(n), nat_divisor_quotient_fn(n))
}

/// The cofactor image of a sublist of the divisor list whose entries inject
/// (under the cofactor map) is itself unique.
define cofactor_image_unique_pred(n: Nat) -> (List[Nat] -> Bool) {
    function(l: List[Nat]) {
        l.is_unique and (forall(d: Nat) { l.contains(d) implies d.divides(n) })
            and Nat.0 < n
            implies map(l, nat_divisor_quotient_fn(n)).is_unique
    }
}

/// The cofactor map composed with `dirichlet_term(g, f, n)` over the divisor
/// list. Inductive predicate driving the composition rewrite.
define cofactor_term_compose_pred(f: Nat -> Nat, g: Nat -> Nat, n: Nat) -> (List[Nat] -> Bool) {
    function(l: List[Nat]) {
        map(l, dirichlet_cofactor_term(f, g, n)) =
            map(map(l, nat_divisor_quotient_fn(n)), dirichlet_term(g, f, n))
    }
}


// dirichlet_assoc.ac
from nat import alt_induction
from list import add_contains_left, add_contains_right, add_contains_or, sum_add, map_add

/// Pairs `(d, e)` with fixed outer divisor `d` and inner divisor
/// `e | divisor_quotient(n, d)`.  These are the natural indices for the
/// right-nested expansion of Dirichlet convolution at `n`.
define right_divisor_pair_block(n: Nat, d: Nat) -> List[Pair[Nat, Nat]] {
    map(divisor_list(divisor_quotient(n, d)), function(e: Nat) {
        Pair.new(d, e)
    })
}

/// Flatten the right-nested divisor-pair blocks over an explicit outer list.
define right_divisor_pair_list_from(n: Nat, outer: List[Nat]) -> List[Pair[Nat, Nat]] {
    match outer {
        List.nil {
            List.nil[Pair[Nat, Nat]]
        }
        List.cons(d, tail) {
            right_divisor_pair_block(n, d) + right_divisor_pair_list_from(n, tail)
        }
    }
}

/// The canonical right-nested divisor-pair list at `n`: all pairs `(d, e)` with
/// `d | n` and `e | divisor_quotient(n, d)`.
define right_divisor_pair_list(n: Nat) -> List[Pair[Nat, Nat]] {
    right_divisor_pair_list_from(n, divisor_list(n))
}

/// The term indexed by a canonical pair `(d, e)`, using the reassociated
/// remaining cofactor `divisor_quotient(n, d * e)`.
define divisor_pair_assoc_term(f: Nat -> Nat, g: Nat -> Nat, h: Nat -> Nat, n: Nat) ->
        (Pair[Nat, Nat] -> Nat) {
    function(p: Pair[Nat, Nat]) {
        f(p.first) * g(p.second) * h(divisor_quotient(n, p.first * p.second))
    }
}

/// A block constructor over an explicit inner divisor list.  This keeps the
/// right-nested expansion proofs bounded over `divisors_up_to` rather than
/// introducing a generic flat-map API.
define right_divisor_pair_block_from(n: Nat, d: Nat, inner: List[Nat]) -> List[Pair[Nat, Nat]] {
    map(inner, function(e: Nat) {
        Pair.new(d, e)
    })
}

/// The right-nested inner summand before rewriting the cofactor into the
/// canonical pair term.
define right_nested_inner_term(f: Nat -> Nat, g: Nat -> Nat, h: Nat -> Nat, n: Nat,
        d: Nat) -> (Nat -> Nat) {
    function(e: Nat) {
        f(d) * dirichlet_term(g, h, divisor_quotient(n, d))(e)
    }
}

/// The canonical pair generated by choosing an inner divisor `d` of a
/// left-nested outer divisor `r`.
define left_divisor_pair_of(r: Nat) -> (Nat -> Pair[Nat, Nat]) {
    function(d: Nat) {
        Pair.new(d, divisor_quotient(r, d))
    }
}

/// Pairs generated from a left-nested divisor choice: an outer divisor `r` and
/// an inner divisor `d | r`, encoded as the canonical pair `(d, r / d)`.
define left_divisor_pair_block(n: Nat, r: Nat) -> List[Pair[Nat, Nat]] {
    map(divisor_list(r), left_divisor_pair_of(r))
}

/// Flatten the left-nested divisor-pair blocks over an explicit outer list.
define left_divisor_pair_list_from(n: Nat, outer: List[Nat]) -> List[Pair[Nat, Nat]] {
    match outer {
        List.nil {
            List.nil[Pair[Nat, Nat]]
        }
        List.cons(r, tail) {
            left_divisor_pair_block(n, r) + left_divisor_pair_list_from(n, tail)
        }
    }
}

/// The canonical left-nested divisor-pair list at `n`: all pairs `(d, r / d)`
/// with `r | n` and `d | r`.
define left_divisor_pair_list(n: Nat) -> List[Pair[Nat, Nat]] {
    left_divisor_pair_list_from(n, divisor_list(n))
}


// unit_fraction.ac
from nat import from_nat
from rat import Rat, reduce
from real import Real
from list import is_lower_bound, is_upper_bound

/// The rational unit fraction with denominator `n`.
/// By the ambient rational convention, the zero denominator gives zero.
define unit_fraction(n: Nat) -> Rat {
    Rat.from_nat(n).inverse
}

/// The real unit fraction with denominator `n`.
/// By the ambient real convention, the zero denominator gives zero.
define real_unit_fraction(n: Nat) -> Real {
    Real.1 / from_nat[Real](n)
}

/// The finite sum of unit fractions with the listed denominators.
define unit_fraction_sum(denominators: List[Nat]) -> Rat {
    sum(map(denominators, unit_fraction))
}

/// The finite real sum of unit fractions with the listed denominators.
define real_unit_fraction_sum(denominators: List[Nat]) -> Real {
    sum(map(denominators, real_unit_fraction))
}

/// True if every listed denominator is positive.
define positive_denominator_list(denominators: List[Nat]) -> Bool {
    match denominators {
        List.nil {
            true
        }
        List.cons(head, tail) {
            Nat.0 < head and positive_denominator_list(tail)
        }
    }
}

/// True if every listed denominator is at least `bound`.
define denominator_list_lower_bound(bound: Nat, denominators: List[Nat]) -> Bool {
    is_lower_bound(denominators, bound)
}

/// True if every listed denominator is at most `bound`.
define denominator_list_upper_bound(bound: Nat, denominators: List[Nat]) -> Bool {
    is_upper_bound(denominators, bound)
}

/// True if the denominators form a distinct positive list.
define egyptian_denominator_list(denominators: List[Nat]) -> Bool {
    denominators.is_unique and positive_denominator_list(denominators)
}

/// True if a rational is a finite sum of distinct unit fractions.
define is_egyptian_fraction(q: Rat) -> Bool {
    exists(denominators: List[Nat]) {
        egyptian_denominator_list(denominators) and q = unit_fraction_sum(denominators)
    }
}

/// True if a rational is represented by distinct unit fractions whose
/// denominators are all at least `bound`.
define is_egyptian_fraction_with_lower_bound(q: Rat, bound: Nat) -> Bool {
    exists(denominators: List[Nat]) {
        egyptian_denominator_list(denominators) and
            denominator_list_lower_bound(bound, denominators) and
            q = unit_fraction_sum(denominators)
    }
}

/// True if a rational has an Egyptian representation using only denominators
/// at least `bound`.
define is_lower_bounded_egyptian_fraction(q: Rat, bound: Nat) -> Bool {
    exists(denominators: List[Nat]) {
        egyptian_denominator_list(denominators) and
        denominator_list_lower_bound(bound, denominators) and
        q = unit_fraction_sum(denominators)
    }
}

/// True if a denominator gives a unit-fraction subtraction step for `q`.
define unit_fraction_greedy_step(q: Rat, n: Nat) -> Bool {
    Nat.0 < n and unit_fraction(n) < q and q - unit_fraction(n) < q
}

/// True if subtracting the unit fraction leaves a positive smaller remainder.
define unit_fraction_bounded_step(q: Rat, n: Nat) -> Bool {
    Nat.0 < n and unit_fraction(n) <= q and
        (q - unit_fraction(n)).is_positive and q - unit_fraction(n) < q
}

/// True if `n` is a positive denominator whose unit fraction is at most `q`.
define unit_fraction_bounded_denominator(q: Rat, n: Nat) -> Bool {
    Nat.0 < n and unit_fraction(n) <= q
}

/// The denominator predicate associated to a fixed rational.
define unit_fraction_bounded_denominator_for(q: Rat) -> (Nat -> Bool) {
    function(n: Nat) {
        unit_fraction_bounded_denominator(q, n)
    }
}

/// True if `n` is the least denominator whose unit fraction is at most `q`.
define unit_fraction_ceiling_step(q: Rat, n: Nat) -> Bool {
    is_min(unit_fraction_bounded_denominator_for(q), n)
}

/// The unreduced numerator of the remainder after subtracting `1/n` from `q`.
define unit_fraction_remainder_raw_num(q: Rat, n: Nat) -> Int {
    q.num * Int.from_nat(n) - q.denom
}

/// The unreduced denominator of the remainder after subtracting `1/n` from `q`.
define unit_fraction_remainder_raw_denom(q: Rat, n: Nat) -> Int {
    q.denom * Int.from_nat(n)
}

/// The natural measure of a rational used by the greedy-step descent.
define unit_fraction_num_measure(q: Rat) -> Nat {
    abs(q.num)
}

/// The natural measure of the unreduced remainder numerator.
define unit_fraction_remainder_raw_num_measure(q: Rat, n: Nat) -> Nat {
    abs(unit_fraction_remainder_raw_num(q, n))
}

/// True when the raw numerator of the remainder is a positive strict descent.
define unit_fraction_raw_num_descends(q: Rat, n: Nat) -> Bool {
    unit_fraction_remainder_raw_num(q, n).is_positive and
        unit_fraction_remainder_raw_num(q, n) < q.num
}

/// A positive denominator gives a positive unit fraction.
theorem unit_fraction_positive(n: Nat) {
    Nat.0 < n implies unit_fraction(n).is_positive
}

/// A positive denominator gives a nonzero unit fraction.
theorem unit_fraction_ne_zero(n: Nat) {
    Nat.0 < n implies unit_fraction(n) != Rat.0
}

/// For a positive denominator, the unit fraction is `1 / n`.
theorem unit_fraction_eq_one_div(n: Nat) {
    Nat.0 < n implies unit_fraction(n) = Rat.1 / Rat.from_nat(n)
}

/// A positive unit fraction is the reduced fraction `1/n`.
theorem unit_fraction_eq_reduce(n: Nat) {
    Nat.0 < n implies unit_fraction(n) = reduce(Int.1, Int.from_nat(n))
}

/// The first unit fraction is one.
theorem unit_fraction_one {
    unit_fraction(Nat.1) = Rat.1
}

/// The zero denominator gives the zero rational.
theorem unit_fraction_zero {
    unit_fraction(Nat.0) = Rat.0
}

/// Multiplying a positive unit fraction by its denominator gives one.
theorem unit_fraction_mul_denominator(n: Nat) {
    Nat.0 < n implies unit_fraction(n) * Rat.from_nat(n) = Rat.1
}

/// A positive rational unit fraction embeds as the matching real unit fraction.
theorem unit_fraction_to_real(n: Nat) {
    Nat.0 < n implies Real.from_rat(unit_fraction(n)) = real_unit_fraction(n)
}

/// Unit fractions are antitone in positive denominators.
theorem unit_fraction_lte_of_lte(m: Nat, n: Nat) {
    Nat.0 < m and m <= n implies unit_fraction(n) <= unit_fraction(m)
}

/// The previous unit fraction is bounded by twice the current unit fraction.
theorem unit_fraction_pred_lte_double(n: Nat) {
    Nat.1 < n implies unit_fraction(n - Nat.1) <= unit_fraction(n) + unit_fraction(n)
}

/// A positive rational below one has a unit-fraction subtraction step.
theorem unit_fraction_greedy_step_exists(q: Rat) {
    q.is_positive and q < Rat.1 implies exists(n: Nat) {
        unit_fraction_greedy_step(q, n)
    }
}

/// A positive rational below one has a bounded unit-fraction subtraction step.
theorem unit_fraction_bounded_step_exists(q: Rat) {
    q.is_positive and q < Rat.1 implies exists(n: Nat) {
        unit_fraction_bounded_step(q, n)
    }
}

/// A bounded step has a positive denominator.
theorem unit_fraction_bounded_step_denominator_positive(q: Rat, n: Nat) {
    unit_fraction_bounded_step(q, n) implies Nat.0 < n
}

/// The unreduced remainder denominator is positive.
theorem unit_fraction_remainder_raw_denom_positive(q: Rat, n: Nat) {
    Nat.0 < n implies unit_fraction_remainder_raw_denom(q, n).is_positive
}

/// A bounded denominator is positive.
theorem unit_fraction_bounded_denominator_positive(q: Rat, n: Nat) {
    unit_fraction_bounded_denominator(q, n) implies Nat.0 < n
}

/// A bounded denominator has a unit fraction below the rational.
theorem unit_fraction_bounded_denominator_lte(q: Rat, n: Nat) {
    unit_fraction_bounded_denominator(q, n) implies unit_fraction(n) <= q
}

/// A bounded denominator gives a nonnegative unreduced remainder numerator.
theorem unit_fraction_bounded_denominator_raw_num_nonnegative(q: Rat, n: Nat) {
    unit_fraction_bounded_denominator(q, n) implies
        Int.0 <= unit_fraction_remainder_raw_num(q, n)
}

/// The remainder after subtracting `1/n` is the raw fraction reduced.
theorem unit_fraction_remainder_eq_raw(q: Rat, n: Nat) {
    Nat.0 < n implies
        q - unit_fraction(n) =
            reduce(unit_fraction_remainder_raw_num(q, n),
                unit_fraction_remainder_raw_denom(q, n))
}

/// The zero raw numerator is exactly a zero reduced remainder.
theorem unit_fraction_remainder_zero_of_raw_num_zero(q: Rat, n: Nat) {
    Nat.0 < n and unit_fraction_remainder_raw_num(q, n) = Int.0 implies
        q - unit_fraction(n) = Rat.0
}

/// A positive bounded-step remainder has positive raw numerator.
theorem unit_fraction_bounded_step_raw_num_positive(q: Rat, n: Nat) {
    unit_fraction_bounded_step(q, n) implies
        unit_fraction_remainder_raw_num(q, n).is_positive
}

/// Every positive rational has a bounded unit-fraction denominator.
theorem unit_fraction_bounded_denominator_exists(q: Rat) {
    q.is_positive implies exists(n: Nat) {
        unit_fraction_bounded_denominator(q, n)
    }
}

/// A canonical ceiling denominator exists for every positive rational.
theorem unit_fraction_ceiling_step_exists(q: Rat) {
    q.is_positive implies exists(n: Nat) {
        unit_fraction_ceiling_step(q, n)
    }
}

/// A canonical ceiling denominator is a bounded denominator.
theorem unit_fraction_ceiling_step_bounded_denominator(q: Rat, n: Nat) {
    unit_fraction_ceiling_step(q, n) implies
        unit_fraction_bounded_denominator(q, n)
}

/// A canonical ceiling denominator is positive.
theorem unit_fraction_ceiling_step_denominator_positive(q: Rat, n: Nat) {
    unit_fraction_ceiling_step(q, n) implies Nat.0 < n
}

/// A canonical ceiling denominator has unit fraction below the rational.
theorem unit_fraction_ceiling_step_lte(q: Rat, n: Nat) {
    unit_fraction_ceiling_step(q, n) implies unit_fraction(n) <= q
}

/// A canonical ceiling denominator is no larger than any bounded denominator.
theorem unit_fraction_ceiling_step_minimal(q: Rat, n: Nat, m: Nat) {
    unit_fraction_ceiling_step(q, n) and unit_fraction_bounded_denominator(q, m)
        implies n <= m
}

/// A canonical ceiling step either finishes exactly or gives a proper bounded step.
theorem unit_fraction_ceiling_step_terminal_or_bounded(q: Rat, n: Nat) {
    unit_fraction_ceiling_step(q, n) implies
        q = unit_fraction(n) or unit_fraction_bounded_step(q, n)
}

/// For a nontrivial canonical ceiling denominator, the previous denominator is too small.
theorem unit_fraction_ceiling_step_pred_lt(q: Rat, n: Nat) {
    unit_fraction_ceiling_step(q, n) and Nat.1 < n
        implies q < unit_fraction(n - Nat.1)
}

/// A canonical ceiling denominator for a rational below one is greater than one.
theorem unit_fraction_ceiling_step_gt_one_of_lt_one(q: Rat, n: Nat) {
    q < Rat.1 and unit_fraction_ceiling_step(q, n) implies Nat.1 < n
}

/// A canonical ceiling denominator below one gives a smaller unreduced numerator.
theorem unit_fraction_ceiling_step_raw_num_lt_num(q: Rat, n: Nat) {
    q < Rat.1 and unit_fraction_ceiling_step(q, n)
        implies unit_fraction_remainder_raw_num(q, n) < q.num
}

/// A positive canonical remainder below one has a smaller reduced numerator.
theorem unit_fraction_ceiling_step_positive_remainder_num_lt_num(q: Rat, n: Nat) {
    q < Rat.1 and unit_fraction_ceiling_step(q, n) and
        (q - unit_fraction(n)).is_positive
        implies (q - unit_fraction(n)).num < q.num
}

/// A nonterminal canonical ceiling step below one has a smaller reduced numerator.
theorem unit_fraction_ceiling_step_remainder_num_lt_num(q: Rat, n: Nat) {
    q < Rat.1 and unit_fraction_ceiling_step(q, n) and q != unit_fraction(n)
        implies (q - unit_fraction(n)).num < q.num
}

/// A nonterminal canonical ceiling step leaves a positive remainder.
theorem unit_fraction_ceiling_step_remainder_positive(q: Rat, n: Nat) {
    unit_fraction_ceiling_step(q, n) and q != unit_fraction(n)
        implies (q - unit_fraction(n)).is_positive
}

/// A nonterminal canonical ceiling step leaves a smaller remainder.
theorem unit_fraction_ceiling_step_remainder_lt_self(q: Rat, n: Nat) {
    unit_fraction_ceiling_step(q, n) and q != unit_fraction(n)
        implies q - unit_fraction(n) < q
}

/// A nonterminal canonical ceiling step below one leaves a remainder below one.
theorem unit_fraction_ceiling_step_remainder_lt_one(q: Rat, n: Nat) {
    q < Rat.1 and unit_fraction_ceiling_step(q, n) and q != unit_fraction(n)
        implies q - unit_fraction(n) < Rat.1
}

/// A canonical ceiling step below one leaves a remainder below the chosen unit
/// fraction.
theorem unit_fraction_ceiling_step_remainder_lt_unit(q: Rat, n: Nat) {
    q < Rat.1 and unit_fraction_ceiling_step(q, n)
        implies q - unit_fraction(n) < unit_fraction(n)
}

/// The next canonical greedy denominator after a canonical step is larger.
theorem unit_fraction_ceiling_step_next_denominator_gt(q: Rat, n: Nat, m: Nat) {
    q < Rat.1 and unit_fraction_ceiling_step(q, n) and
        unit_fraction_ceiling_step(q - unit_fraction(n), m)
        implies n < m
}

/// The next canonical greedy denominator is lower-bounded by the successor of
/// the current denominator.
theorem unit_fraction_ceiling_step_next_denominator_lower_bound(q: Rat, n: Nat,
    m: Nat) {
    q < Rat.1 and unit_fraction_ceiling_step(q, n) and
        unit_fraction_ceiling_step(q - unit_fraction(n), m)
        implies n.suc <= m
}

/// A canonical bounded step below one gives strict descent of the raw numerator.
theorem unit_fraction_ceiling_step_raw_num_descends(q: Rat, n: Nat) {
    q.is_positive and q < Rat.1 and unit_fraction_ceiling_step(q, n) and
        unit_fraction_bounded_step(q, n)
        implies unit_fraction_raw_num_descends(q, n)
}

/// A canonical bounded step below one strictly decreases the natural numerator measure.
theorem unit_fraction_ceiling_step_raw_num_measure_lt(q: Rat, n: Nat) {
    q.is_positive and q < Rat.1 and unit_fraction_ceiling_step(q, n) and
        unit_fraction_bounded_step(q, n)
        implies unit_fraction_remainder_raw_num_measure(q, n) <
            unit_fraction_num_measure(q)
}

/// The empty unit-fraction sum is zero.
theorem unit_fraction_sum_nil {
    unit_fraction_sum(List.nil[Nat]) = Rat.0
}

/// The empty real unit-fraction sum is zero.
theorem real_unit_fraction_sum_nil {
    real_unit_fraction_sum(List.nil[Nat]) = Real.0
}

/// Consing a denominator adds its unit fraction to the front of the sum.
theorem unit_fraction_sum_cons(n: Nat, denominators: List[Nat]) {
    unit_fraction_sum(List.cons(n, denominators)) =
        unit_fraction(n) + unit_fraction_sum(denominators)
}

/// Consing a denominator adds its real unit fraction to the front of the sum.
theorem real_unit_fraction_sum_cons(n: Nat, denominators: List[Nat]) {
    real_unit_fraction_sum(List.cons(n, denominators)) =
        real_unit_fraction(n) + real_unit_fraction_sum(denominators)
}

/// A singleton unit-fraction sum is that unit fraction.
theorem unit_fraction_sum_singleton(n: Nat) {
    unit_fraction_sum(List.singleton(n)) = unit_fraction(n)
}

/// The unit-fraction sum of a concatenation splits as a sum.
theorem unit_fraction_sum_append(left: List[Nat], right: List[Nat]) {
    unit_fraction_sum(left + right) = unit_fraction_sum(left) + unit_fraction_sum(right)
}

/// A finite sum over positive denominators embeds into the matching real sum.
theorem unit_fraction_sum_to_real(denominators: List[Nat]) {
    positive_denominator_list(denominators) implies
        Real.from_rat(unit_fraction_sum(denominators)) =
            real_unit_fraction_sum(denominators)
}

/// A unit-fraction sum over positive denominators is nonnegative.
theorem unit_fraction_sum_nonnegative(denominators: List[Nat]) {
    positive_denominator_list(denominators) implies Rat.0 <= unit_fraction_sum(denominators)
}

/// Consing a positive denominator gives a positive unit-fraction sum.
theorem unit_fraction_sum_cons_positive(n: Nat, denominators: List[Nat]) {
    Nat.0 < n and positive_denominator_list(denominators)
        implies unit_fraction_sum(List.cons(n, denominators)).is_positive
}

/// The empty list has positive denominators.
theorem positive_denominator_list_nil {
    positive_denominator_list(List.nil[Nat])
}

/// Consing a positive denominator onto a positive denominator list preserves positivity.
theorem positive_denominator_list_cons(n: Nat, denominators: List[Nat]) {
    Nat.0 < n and positive_denominator_list(denominators)
        implies positive_denominator_list(List.cons(n, denominators))
}

/// A nonempty positive denominator list has a positive head.
theorem positive_denominator_list_head(n: Nat, denominators: List[Nat]) {
    positive_denominator_list(List.cons(n, denominators)) implies Nat.0 < n
}

/// The tail of a nonempty positive denominator list has positive denominators.
theorem positive_denominator_list_tail(n: Nat, denominators: List[Nat]) {
    positive_denominator_list(List.cons(n, denominators))
        implies positive_denominator_list(denominators)
}

/// Positive denominator lists remain positive after append.
theorem positive_denominator_list_append(left: List[Nat], right: List[Nat]) {
    positive_denominator_list(left) and positive_denominator_list(right)
        implies positive_denominator_list(left + right)
}

/// The empty list satisfies every lower bound.
theorem denominator_list_lower_bound_nil(bound: Nat) {
    denominator_list_lower_bound(bound, List.nil[Nat])
}

/// The empty list satisfies every upper bound.
theorem denominator_list_upper_bound_nil(bound: Nat) {
    denominator_list_upper_bound(bound, List.nil[Nat])
}

/// Consing a denominator above the lower bound preserves the lower bound.
theorem denominator_list_lower_bound_cons(bound: Nat, n: Nat, denominators: List[Nat]) {
    bound <= n and denominator_list_lower_bound(bound, denominators)
        implies denominator_list_lower_bound(bound, List.cons(n, denominators))
}

/// Consing a denominator below the upper bound preserves the upper bound.
theorem denominator_list_upper_bound_cons(bound: Nat, n: Nat, denominators: List[Nat]) {
    n <= bound and denominator_list_upper_bound(bound, denominators)
        implies denominator_list_upper_bound(bound, List.cons(n, denominators))
}

/// A nonempty lower-bounded denominator list has a lower-bounded head.
theorem denominator_list_lower_bound_head(bound: Nat, n: Nat, denominators: List[Nat]) {
    denominator_list_lower_bound(bound, List.cons(n, denominators)) implies bound <= n
}

/// The tail of a nonempty lower-bounded denominator list is lower-bounded.
theorem denominator_list_lower_bound_tail(bound: Nat, n: Nat, denominators: List[Nat]) {
    denominator_list_lower_bound(bound, List.cons(n, denominators))
        implies denominator_list_lower_bound(bound, denominators)
}

/// A smaller lower bound is also a lower bound for the same denominators.
theorem denominator_list_lower_bound_monotone(bound: Nat, smaller: Nat,
    denominators: List[Nat]) {
    denominator_list_lower_bound(bound, denominators) and smaller <= bound
        implies denominator_list_lower_bound(smaller, denominators)
}

/// A tail bounded below by `n.suc` is also bounded below by `n`.
theorem denominator_list_lower_bound_suc_imp_lower_bound(n: Nat,
    denominators: List[Nat]) {
    denominator_list_lower_bound(n.suc, denominators)
        implies denominator_list_lower_bound(n, denominators)
}

/// Consing `n` onto a tail bounded below by `n.suc` preserves lower bound `n`.
theorem denominator_list_lower_bound_cons_self_of_suc(n: Nat,
    denominators: List[Nat]) {
    denominator_list_lower_bound(n.suc, denominators)
        implies denominator_list_lower_bound(n, List.cons(n, denominators))
}

/// Positive denominator lists have lower bound one.
theorem positive_denominator_list_lower_bound_one(denominators: List[Nat]) {
    positive_denominator_list(denominators) implies
        denominator_list_lower_bound(Nat.1, denominators)
}

/// Every member of a positive denominator list is positive.
theorem positive_denominator_list_contains_positive(denominators: List[Nat], n: Nat) {
    positive_denominator_list(denominators) and denominators.contains(n) implies Nat.0 < n
}

/// A lower bound above `n` makes `n` fresh for the denominator list.
theorem denominator_list_lower_bound_suc_not_contains(n: Nat,
    denominators: List[Nat]) {
    denominator_list_lower_bound(n.suc, denominators)
        implies not denominators.contains(n)
}

/// A nonempty upper-bounded denominator list has an upper-bounded head.
theorem denominator_list_upper_bound_head(bound: Nat, n: Nat, denominators: List[Nat]) {
    denominator_list_upper_bound(bound, List.cons(n, denominators)) implies n <= bound
}

/// The tail of a nonempty upper-bounded denominator list is upper-bounded.
theorem denominator_list_upper_bound_tail(bound: Nat, n: Nat, denominators: List[Nat]) {
    denominator_list_upper_bound(bound, List.cons(n, denominators))
        implies denominator_list_upper_bound(bound, denominators)
}

/// Lower bounds are preserved by appending denominator lists.
theorem denominator_list_lower_bound_append(bound: Nat, left: List[Nat], right: List[Nat]) {
    denominator_list_lower_bound(bound, left) and denominator_list_lower_bound(bound, right)
        implies denominator_list_lower_bound(bound, left + right)
}

/// Upper bounds are preserved by appending denominator lists.
theorem denominator_list_upper_bound_append(bound: Nat, left: List[Nat], right: List[Nat]) {
    denominator_list_upper_bound(bound, left) and denominator_list_upper_bound(bound, right)
        implies denominator_list_upper_bound(bound, left + right)
}

/// Lower bounds are preserved by filtering denominator lists.
theorem denominator_list_lower_bound_filter(bound: Nat, denominators: List[Nat],
    pred: Nat -> Bool) {
    denominator_list_lower_bound(bound, denominators)
        implies denominator_list_lower_bound(bound, denominators.filter(pred))
}

/// Upper bounds are preserved by filtering denominator lists.
theorem denominator_list_upper_bound_filter(bound: Nat, denominators: List[Nat],
    pred: Nat -> Bool) {
    denominator_list_upper_bound(bound, denominators)
        implies denominator_list_upper_bound(bound, denominators.filter(pred))
}

/// A positive singleton denominator list has positive denominators.
theorem positive_denominator_list_singleton(n: Nat) {
    Nat.0 < n implies positive_denominator_list(List.singleton(n))
}

/// The empty list is a distinct positive denominator list.
theorem egyptian_denominator_list_nil {
    egyptian_denominator_list(List.nil[Nat])
}

/// An Egyptian denominator list has unique denominators.
theorem egyptian_denominator_list_unique(denominators: List[Nat]) {
    egyptian_denominator_list(denominators) implies denominators.is_unique
}

/// An Egyptian denominator list has positive denominators.
theorem egyptian_denominator_list_positive(denominators: List[Nat]) {
    egyptian_denominator_list(denominators) implies
        positive_denominator_list(denominators)
}

/// A positive singleton denominator is a distinct positive denominator list.
theorem egyptian_denominator_list_singleton(n: Nat) {
    Nat.0 < n implies egyptian_denominator_list(List.singleton(n))
}

/// Consing a fresh positive denominator preserves Egyptian denominator lists.
theorem egyptian_denominator_list_cons(n: Nat, denominators: List[Nat]) {
    Nat.0 < n and egyptian_denominator_list(denominators) and
        not denominators.contains(n)
        implies egyptian_denominator_list(List.cons(n, denominators))
}

/// Appending disjoint Egyptian denominator lists gives an Egyptian denominator list.
theorem egyptian_denominator_list_append(left: List[Nat], right: List[Nat]) {
    egyptian_denominator_list(left) and egyptian_denominator_list(right) and
        (forall(n: Nat) { not (left.contains(n) and right.contains(n)) })
        implies egyptian_denominator_list(left + right)
}

/// Zero is represented by the empty sum of unit fractions.
theorem is_egyptian_fraction_zero {
    is_egyptian_fraction(Rat.0)
}

/// A lower-bounded Egyptian fraction is an Egyptian fraction.
theorem is_egyptian_fraction_with_lower_bound_imp_egyptian(q: Rat, bound: Nat) {
    is_egyptian_fraction_with_lower_bound(q, bound) implies is_egyptian_fraction(q)
}

/// Zero is represented by the empty sum with any denominator lower bound.
theorem is_egyptian_fraction_with_lower_bound_zero(bound: Nat) {
    is_egyptian_fraction_with_lower_bound(Rat.0, bound)
}

/// A unit-fraction sum over Egyptian denominators is an Egyptian fraction.
theorem unit_fraction_sum_is_egyptian(denominators: List[Nat]) {
    egyptian_denominator_list(denominators)
        implies is_egyptian_fraction(unit_fraction_sum(denominators))
}

/// A unit-fraction sum over lower-bounded Egyptian denominators is a
/// lower-bounded Egyptian fraction.
theorem unit_fraction_sum_is_egyptian_with_lower_bound(denominators: List[Nat],
    bound: Nat) {
    egyptian_denominator_list(denominators) and
        denominator_list_lower_bound(bound, denominators)
        implies is_egyptian_fraction_with_lower_bound(unit_fraction_sum(denominators),
            bound)
}

/// A smaller denominator lower bound preserves a lower-bounded Egyptian
/// representation.
theorem is_egyptian_fraction_with_lower_bound_monotone(q: Rat, bound: Nat,
    smaller: Nat) {
    is_egyptian_fraction_with_lower_bound(q, bound) and smaller <= bound
        implies is_egyptian_fraction_with_lower_bound(q, smaller)
}

/// True when two Egyptian representations may be added by appending their
/// disjoint denominator lists.
define egyptian_fraction_add_disjoint_hyp(q: Rat, r: Rat, left: List[Nat],
    right: List[Nat]) -> Bool {
    egyptian_denominator_list(left) and egyptian_denominator_list(right) and
    (forall(n: Nat) { not (left.contains(n) and right.contains(n)) }) and
    q = unit_fraction_sum(left) and r = unit_fraction_sum(right)
}

/// Adding two disjoint Egyptian representations gives an Egyptian fraction.
theorem egyptian_fraction_add_disjoint(q: Rat, r: Rat, left: List[Nat], right: List[Nat]) {
    egyptian_fraction_add_disjoint_hyp(q, r, left, right) implies is_egyptian_fraction(q + r)
}

/// Adding two disjoint lower-bounded Egyptian representations gives a
/// lower-bounded Egyptian fraction.
theorem egyptian_fraction_add_disjoint_lower_bound(q: Rat, r: Rat, bound: Nat,
    left: List[Nat], right: List[Nat]) {
    egyptian_denominator_list(left) and egyptian_denominator_list(right) and
        (forall(n: Nat) { not (left.contains(n) and right.contains(n)) }) and
        denominator_list_lower_bound(bound, left) and
        denominator_list_lower_bound(bound, right) and
        q = unit_fraction_sum(left) and r = unit_fraction_sum(right)
        implies is_egyptian_fraction_with_lower_bound(q + r, bound)
}

/// Adding a fresh positive denominator to an Egyptian representation gives an
/// Egyptian fraction.
theorem egyptian_fraction_add_fresh_unit(q: Rat, n: Nat, denominators: List[Nat]) {
    Nat.0 < n and egyptian_denominator_list(denominators) and
        not denominators.contains(n) and q = unit_fraction_sum(denominators)
        implies is_egyptian_fraction(unit_fraction(n) + q)
}

/// Adding a denominator below a represented lower-bound tail is fresh.
theorem egyptian_fraction_add_lower_bound_unit(q: Rat, n: Nat,
    denominators: List[Nat]) {
    Nat.0 < n and egyptian_denominator_list(denominators) and
        denominator_list_lower_bound(n.suc, denominators) and
        q = unit_fraction_sum(denominators)
        implies is_egyptian_fraction(unit_fraction(n) + q)
}

/// Adding a denominator below a represented lower-bound tail preserves a
/// lower-bounded Egyptian representation at the current denominator.
theorem lower_bounded_egyptian_fraction_add_lower_bound_unit(q: Rat, n: Nat,
    denominators: List[Nat]) {
    Nat.0 < n and egyptian_denominator_list(denominators) and
        denominator_list_lower_bound(n.suc, denominators) and
        q = unit_fraction_sum(denominators)
        implies is_lower_bounded_egyptian_fraction(unit_fraction(n) + q, n)
}

/// A lower-bounded Egyptian representation is in particular Egyptian.
theorem lower_bounded_egyptian_fraction_is_egyptian(q: Rat, bound: Nat) {
    is_lower_bounded_egyptian_fraction(q, bound) implies is_egyptian_fraction(q)
}

/// A fresh Egyptian representation of a bounded-step remainder represents the
/// original rational.
theorem egyptian_fraction_of_bounded_step_remainder(q: Rat, n: Nat,
    denominators: List[Nat]) {
    Nat.0 < n and unit_fraction_bounded_step(q, n) and
        egyptian_denominator_list(denominators) and
        not denominators.contains(n) and
        q - unit_fraction(n) = unit_fraction_sum(denominators)
        implies is_egyptian_fraction(q)
}

/// A lower-bounded Egyptian representation of a unit-fraction remainder
/// represents the original rational.
theorem egyptian_fraction_of_lower_bound_remainder(q: Rat, n: Nat,
    denominators: List[Nat]) {
    Nat.0 < n and egyptian_denominator_list(denominators) and
        denominator_list_lower_bound(n.suc, denominators) and
        q - unit_fraction(n) = unit_fraction_sum(denominators)
        implies is_egyptian_fraction(q)
}

/// Every positive unit fraction is an Egyptian fraction.
theorem unit_fraction_is_egyptian(n: Nat) {
    Nat.0 < n implies is_egyptian_fraction(unit_fraction(n))
}

/// A positive unit fraction has a lower-bounded Egyptian representation
/// whenever its denominator is above the bound.
theorem unit_fraction_is_egyptian_with_lower_bound(n: Nat, bound: Nat) {
    Nat.0 < n and bound <= n
        implies is_egyptian_fraction_with_lower_bound(unit_fraction(n), bound)
}

/// A lower-bounded Egyptian representation of a remainder extends to a
/// lower-bounded representation of the original rational.
theorem egyptian_fraction_with_lower_bound_of_lower_bound_remainder(q: Rat,
    n: Nat, denominators: List[Nat]) {
    Nat.0 < n and egyptian_denominator_list(denominators) and
        denominator_list_lower_bound(n.suc, denominators) and
        q - unit_fraction(n) = unit_fraction_sum(denominators)
        implies is_egyptian_fraction_with_lower_bound(q, n)
}

/// A lower-bounded Egyptian representation of a remainder extends to a
/// lower-bounded representation of the original rational.
theorem egyptian_fraction_with_lower_bound_of_remainder(q: Rat, n: Nat) {
    Nat.0 < n and
        is_egyptian_fraction_with_lower_bound(q - unit_fraction(n), n.suc)
        implies is_egyptian_fraction_with_lower_bound(q, n)
}

/// A positive rational below a unit fraction has a lower-bounded Egyptian
/// representation, with the numerator measure supplied explicitly.
theorem positive_lt_unit_fraction_is_egyptian_with_lower_bound_of_num(q: Rat,
    prev: Nat, k: Nat) {
    q.is_positive and q < unit_fraction(prev) and q.num = Int.from_nat(k.suc)
        implies is_egyptian_fraction_with_lower_bound(q, prev.suc)
}

/// A positive rational below a unit fraction has an Egyptian representation
/// whose denominators are all above the predecessor denominator.
theorem positive_lt_unit_fraction_is_egyptian_with_lower_bound(q: Rat, prev: Nat) {
    q.is_positive and q < unit_fraction(prev)
        implies is_egyptian_fraction_with_lower_bound(q, prev.suc)
}

/// A positive rational below one has an Egyptian representation whose
/// denominators are at least two.
theorem positive_lt_one_is_egyptian_with_lower_bound_two(q: Rat) {
    q.is_positive and q < Rat.1
        implies is_egyptian_fraction_with_lower_bound(q, Nat.2)
}

/// A positive rational below one is an Egyptian fraction.
theorem positive_lt_one_is_egyptian(q: Rat) {
    q.is_positive and q < Rat.1 implies is_egyptian_fraction(q)
}


// four_squares.ac
/// A natural number is a sum of four squares if it has four natural witnesses.
define is_sum_four_squares(n: Nat) -> Bool {
    exists(a: Nat, b: Nat, c: Nat, d: Nat) {
        n = a * a + b * b + c * c + d * d
    }
}


// continued_fraction.ac
/// True if every coefficient in a continued-fraction tail is positive.
define positive_continued_fraction_tail(coefficients: List[Nat]) -> Bool {
    match coefficients {
        List.nil {
            true
        }
        List.cons(head, tail) {
            Nat.0 < head and positive_continued_fraction_tail(tail)
        }
    }
}

/// True if the coefficients form a finite simple continued fraction.
define finite_continued_fraction_coefficients(coefficients: List[Nat]) -> Bool {
    match coefficients {
        List.nil {
            false
        }
        List.cons(head, tail) {
            positive_continued_fraction_tail(tail)
        }
    }
}

/// A finite simple continued fraction.
structure ContinuedFraction {
    /// The coefficients of the continued fraction.
    coefficients: List[Nat]
} constraint {
    finite_continued_fraction_coefficients(coefficients)
}

/// The rational value of a finite simple continued fraction.
define continued_fraction_value(coefficients: List[Nat]) -> Rat {
    match coefficients {
        List.nil {
            Rat.0
        }
        List.cons(head, tail) {
            Rat.from_nat(head) + continued_fraction_value(tail).inverse
        }
    }
}

/// The forward recurrence state for a finite continuant.
define continuant_state(coefficients: List[Nat], previous: Nat, current: Nat) -> Nat {
    match coefficients {
        List.nil {
            current
        }
        List.cons(head, tail) {
            continuant_state(tail, current, current * head + previous)
        }
    }
}

/// The continuant associated to a finite coefficient list.
define continuant(coefficients: List[Nat]) -> Nat {
    continuant_state(coefficients, Nat.0, Nat.1)
}

/// The numerator of the finite continued-fraction convergent.
define continued_fraction_numerator(coefficients: List[Nat]) -> Nat {
    match coefficients {
        List.nil {
            Nat.0
        }
        List.cons(head, tail) {
            continuant(coefficients)
        }
    }
}

/// The denominator of the finite continued-fraction convergent.
define continued_fraction_denominator(coefficients: List[Nat]) -> Nat {
    match coefficients {
        List.nil {
            Nat.1
        }
        List.cons(head, tail) {
            continuant(tail)
        }
    }
}

/// The numerator-denominator recurrence state for finite convergents.
define continued_fraction_convergent_state(coefficients: List[Nat],
    previous_numerator: Nat, current_numerator: Nat,
    previous_denominator: Nat, current_denominator: Nat) -> Pair[Nat, Nat] {
    match coefficients {
        List.nil {
            Pair.new(current_numerator, current_denominator)
        }
        List.cons(head, tail) {
            continued_fraction_convergent_state(tail,
                current_numerator,
                current_numerator * head + previous_numerator,
                current_denominator,
                current_denominator * head + previous_denominator)
        }
    }
}

/// The numerator-denominator pair for a finite continued fraction.
define continued_fraction_convergent(coefficients: List[Nat]) -> Pair[Nat, Nat] {
    match coefficients {
        List.nil {
            Pair.new(Nat.0, Nat.1)
        }
        List.cons(head, tail) {
            continued_fraction_convergent_state(coefficients, Nat.0, Nat.1, Nat.1, Nat.0)
        }
    }
}

/// The empty tail is positive.
theorem positive_continued_fraction_tail_nil {
    positive_continued_fraction_tail(List.nil[Nat])
}

/// The head of a positive continued-fraction tail is positive.
theorem positive_continued_fraction_tail_cons_head(head: Nat, tail: List[Nat]) {
    positive_continued_fraction_tail(List.cons(head, tail)) implies Nat.0 < head
}

/// The tail of a positive continued-fraction tail is positive.
theorem positive_continued_fraction_tail_cons_tail(head: Nat, tail: List[Nat]) {
    positive_continued_fraction_tail(List.cons(head, tail))
        implies positive_continued_fraction_tail(tail)
}

/// Consing a positive coefficient onto a positive tail gives a positive tail.
theorem positive_continued_fraction_tail_cons_intro(head: Nat, tail: List[Nat]) {
    Nat.0 < head and positive_continued_fraction_tail(tail)
        implies positive_continued_fraction_tail(List.cons(head, tail))
}

/// A singleton tail is positive exactly when its coefficient is positive.
theorem positive_continued_fraction_tail_singleton_intro(head: Nat) {
    Nat.0 < head implies positive_continued_fraction_tail(List.cons(head, List.nil[Nat]))
}

/// The coefficient in a positive singleton tail is positive.
theorem positive_continued_fraction_tail_singleton_positive(head: Nat) {
    positive_continued_fraction_tail(List.cons(head, List.nil[Nat])) implies Nat.0 < head
}

/// A two-element tail is positive when both coefficients are positive.
theorem positive_continued_fraction_tail_pair_intro(head: Nat, next: Nat) {
    Nat.0 < head and Nat.0 < next
        implies positive_continued_fraction_tail(List.cons(head, List.cons(next, List.nil[Nat])))
}

/// A singleton list of coefficients is a finite simple continued fraction.
theorem finite_continued_fraction_coefficients_singleton(head: Nat) {
    finite_continued_fraction_coefficients(List.cons(head, List.nil[Nat]))
}

/// The continued fraction with one coefficient.
let continued_fraction_singleton(head: Nat) -> cf: ContinuedFraction satisfy {
    ContinuedFraction.new(List.cons(head, List.nil[Nat])) = Option.some(cf)
}

attributes ContinuedFraction {
    /// The continued fraction with one coefficient.
    let singleton = continued_fraction_singleton

    /// The rational value of the continued fraction.
    define value(self) -> Rat {
        continued_fraction_value(self.coefficients)
    }

    /// The continuant associated to the continued fraction.
    define continuant(self) -> Nat {
        continuant(self.coefficients)
    }

    /// The numerator of the finite convergent.
    define numerator(self) -> Nat {
        continued_fraction_numerator(self.coefficients)
    }

    /// The denominator of the finite convergent.
    define denominator(self) -> Nat {
        continued_fraction_denominator(self.coefficients)
    }

    /// The numerator-denominator pair of the finite convergent.
    define convergent(self) -> Pair[Nat, Nat] {
        continued_fraction_convergent(self.coefficients)
    }
}

/// A continued fraction has valid coefficients.
theorem continued_fraction_coefficients_valid(cf: ContinuedFraction) {
    finite_continued_fraction_coefficients(cf.coefficients)
}

/// Rebuilding a continued fraction from its coefficients gives the original fraction.
theorem continued_fraction_new_self(cf: ContinuedFraction) {
    ContinuedFraction.new(cf.coefficients) = Option.some(cf)
}

/// The value method is the coefficient-list value.
theorem continued_fraction_value_eq_coefficients_value(cf: ContinuedFraction) {
    cf.value = continued_fraction_value(cf.coefficients)
}

/// The continuant method is the coefficient-list continuant.
theorem continued_fraction_continuant_eq_coefficients_continuant(cf: ContinuedFraction) {
    cf.continuant = continuant(cf.coefficients)
}

/// The numerator method is the coefficient-list numerator.
theorem continued_fraction_numerator_eq_coefficients_numerator(cf: ContinuedFraction) {
    cf.numerator = continued_fraction_numerator(cf.coefficients)
}

/// The denominator method is the coefficient-list denominator.
theorem continued_fraction_denominator_eq_coefficients_denominator(cf: ContinuedFraction) {
    cf.denominator = continued_fraction_denominator(cf.coefficients)
}

/// The convergent method is the coefficient-list convergent.
theorem continued_fraction_convergent_eq_coefficients_convergent(cf: ContinuedFraction) {
    cf.convergent = continued_fraction_convergent(cf.coefficients)
}

/// The singleton constructor has the expected coefficient list.
theorem continued_fraction_singleton_coefficients(head: Nat) {
    ContinuedFraction.singleton(head).coefficients = List.cons(head, List.nil[Nat])
}

/// Coefficient validity unfolds over a nonempty list.
theorem finite_continued_fraction_coefficients_cons(head: Nat, tail: List[Nat]) {
    finite_continued_fraction_coefficients(List.cons(head, tail)) =
        positive_continued_fraction_tail(tail)
}

/// A valid nonempty coefficient list has a positive tail.
theorem finite_continued_fraction_coefficients_cons_tail(head: Nat, tail: List[Nat]) {
    finite_continued_fraction_coefficients(List.cons(head, tail))
        implies positive_continued_fraction_tail(tail)
}

/// A positive tail gives a valid nonempty coefficient list.
theorem finite_continued_fraction_coefficients_cons_intro(head: Nat, tail: List[Nat]) {
    positive_continued_fraction_tail(tail)
        implies finite_continued_fraction_coefficients(List.cons(head, tail))
}

/// A two-term list with a positive second coefficient is valid.
theorem finite_continued_fraction_coefficients_pair_intro(head: Nat, next: Nat) {
    Nat.0 < next implies
        finite_continued_fraction_coefficients(List.cons(head, List.cons(next, List.nil[Nat])))
}

/// A valid two-term list has a positive second coefficient.
theorem finite_continued_fraction_coefficients_pair_tail_positive(head: Nat, next: Nat) {
    finite_continued_fraction_coefficients(List.cons(head, List.cons(next, List.nil[Nat])))
        implies Nat.0 < next
}

/// A three-term list with positive tail coefficients is valid.
theorem finite_continued_fraction_coefficients_triple_intro(head: Nat, next: Nat,
    third: Nat) {
    Nat.0 < next and Nat.0 < third implies
        finite_continued_fraction_coefficients(
            List.cons(head, List.cons(next, List.cons(third, List.nil[Nat]))))
}

/// A valid three-term coefficient list has a positive second coefficient.
theorem finite_continued_fraction_coefficients_triple_second_positive(head: Nat,
    next: Nat, third: Nat) {
    finite_continued_fraction_coefficients(
        List.cons(head, List.cons(next, List.cons(third, List.nil[Nat]))))
        implies Nat.0 < next
}

/// A valid three-term coefficient list has a positive third coefficient.
theorem finite_continued_fraction_coefficients_triple_third_positive(head: Nat,
    next: Nat, third: Nat) {
    finite_continued_fraction_coefficients(
        List.cons(head, List.cons(next, List.cons(third, List.nil[Nat]))))
        implies Nat.0 < third
}

/// The empty coefficient list has value zero.
theorem continued_fraction_value_nil {
    continued_fraction_value(List.nil[Nat]) = Rat.0
}

/// The value of a nonempty list unfolds by adding the reciprocal of the tail.
theorem continued_fraction_value_cons(head: Nat, tail: List[Nat]) {
    continued_fraction_value(List.cons(head, tail)) =
        Rat.from_nat(head) + continued_fraction_value(tail).inverse
}

/// A singleton continued fraction has the value of its single coefficient.
theorem continued_fraction_value_singleton(head: Nat) {
    continued_fraction_value(List.cons(head, List.nil[Nat])) = Rat.from_nat(head)
}

/// A singleton continued fraction has the value of its single coefficient.
theorem continued_fraction_singleton_value(head: Nat) {
    ContinuedFraction.singleton(head).value = Rat.from_nat(head)
}

/// The continuant of the empty list is one.
theorem continuant_nil {
    continuant(List.nil[Nat]) = Nat.1
}

/// The empty state returns the current continuant.
theorem continuant_state_nil(previous: Nat, current: Nat) {
    continuant_state(List.nil[Nat], previous, current) = current
}

/// A nonempty state advances the continuant recurrence by one coefficient.
theorem continuant_state_cons(head: Nat, tail: List[Nat], previous: Nat, current: Nat) {
    continuant_state(List.cons(head, tail), previous, current) =
        continuant_state(tail, current, current * head + previous)
}

/// The continuant of a singleton list is its single coefficient.
theorem continuant_singleton(head: Nat) {
    continuant(List.cons(head, List.nil[Nat])) = head
}

/// A singleton continued fraction has continuant equal to its coefficient.
theorem continued_fraction_singleton_continuant(head: Nat) {
    ContinuedFraction.singleton(head).continuant = head
}

/// The continuant of a two-element list is `head * next + 1`.
theorem continuant_pair(head: Nat, next: Nat) {
    continuant(List.cons(head, List.cons(next, List.nil[Nat]))) =
        head * next + Nat.1
}

/// The continuant of a three-element list follows the third recurrence step.
theorem continuant_triple(head: Nat, next: Nat, third: Nat) {
    continuant(List.cons(head, List.cons(next, List.cons(third, List.nil[Nat])))) =
        (head * next + Nat.1) * third + head
}

/// The empty continued fraction has numerator zero.
theorem continued_fraction_numerator_nil {
    continued_fraction_numerator(List.nil[Nat]) = Nat.0
}

/// A singleton continued fraction has numerator equal to its coefficient.
theorem continued_fraction_numerator_singleton(head: Nat) {
    continued_fraction_numerator(List.cons(head, List.nil[Nat])) = head
}

/// A singleton continued fraction has numerator equal to its coefficient.
theorem continued_fraction_singleton_numerator(head: Nat) {
    ContinuedFraction.singleton(head).numerator = head
}

/// A two-term continued fraction has numerator `head * next + 1`.
theorem continued_fraction_numerator_pair(head: Nat, next: Nat) {
    continued_fraction_numerator(List.cons(head, List.cons(next, List.nil[Nat]))) =
        head * next + Nat.1
}

/// A three-term continued fraction has numerator given by the third continuant.
theorem continued_fraction_numerator_triple(head: Nat, next: Nat, third: Nat) {
    continued_fraction_numerator(
        List.cons(head, List.cons(next, List.cons(third, List.nil[Nat])))) =
        (head * next + Nat.1) * third + head
}

/// The empty continued fraction has denominator one.
theorem continued_fraction_denominator_nil {
    continued_fraction_denominator(List.nil[Nat]) = Nat.1
}

/// The empty convergent is the conventional pair `0/1`.
theorem continued_fraction_convergent_nil {
    continued_fraction_convergent(List.nil[Nat]) = Pair.new(Nat.0, Nat.1)
}

/// The empty convergent has numerator zero.
theorem continued_fraction_convergent_nil_first {
    continued_fraction_convergent(List.nil[Nat]).first = Nat.0
}

/// The empty convergent has denominator one.
theorem continued_fraction_convergent_nil_second {
    continued_fraction_convergent(List.nil[Nat]).second = Nat.1
}

/// The empty convergent has positive denominator.
theorem continued_fraction_convergent_nil_second_positive {
    Nat.0 < continued_fraction_convergent(List.nil[Nat]).second
}

/// The empty convergent first projection matches the numerator definition.
theorem continued_fraction_convergent_nil_first_eq_numerator {
    continued_fraction_convergent(List.nil[Nat]).first =
        continued_fraction_numerator(List.nil[Nat])
}

/// The empty convergent second projection matches the denominator definition.
theorem continued_fraction_convergent_nil_second_eq_denominator {
    continued_fraction_convergent(List.nil[Nat]).second =
        continued_fraction_denominator(List.nil[Nat])
}

/// The empty convergent state returns the current numerator and denominator.
theorem continued_fraction_convergent_state_nil(previous_numerator: Nat,
    current_numerator: Nat, previous_denominator: Nat, current_denominator: Nat) {
    continued_fraction_convergent_state(List.nil[Nat], previous_numerator,
        current_numerator, previous_denominator, current_denominator) =
        Pair.new(current_numerator, current_denominator)
}

/// The first projection of an empty convergent state is the current numerator.
theorem continued_fraction_convergent_state_nil_first(previous_numerator: Nat,
    current_numerator: Nat, previous_denominator: Nat, current_denominator: Nat) {
    continued_fraction_convergent_state(List.nil[Nat], previous_numerator,
        current_numerator, previous_denominator, current_denominator).first =
        current_numerator
}

/// The second projection of an empty convergent state is the current denominator.
theorem continued_fraction_convergent_state_nil_second(previous_numerator: Nat,
    current_numerator: Nat, previous_denominator: Nat, current_denominator: Nat) {
    continued_fraction_convergent_state(List.nil[Nat], previous_numerator,
        current_numerator, previous_denominator, current_denominator).second =
        current_denominator
}

/// A nonempty convergent state advances both numerator and denominator recurrences.
theorem continued_fraction_convergent_state_cons(head: Nat, tail: List[Nat],
    previous_numerator: Nat, current_numerator: Nat,
    previous_denominator: Nat, current_denominator: Nat) {
    continued_fraction_convergent_state(List.cons(head, tail), previous_numerator,
        current_numerator, previous_denominator, current_denominator) =
        continued_fraction_convergent_state(tail,
            current_numerator,
            current_numerator * head + previous_numerator,
            current_denominator,
            current_denominator * head + previous_denominator)
}

/// The first projection of a nonempty state follows the numerator recurrence.
theorem continued_fraction_convergent_state_cons_first(head: Nat, tail: List[Nat],
    previous_numerator: Nat, current_numerator: Nat,
    previous_denominator: Nat, current_denominator: Nat) {
    continued_fraction_convergent_state(List.cons(head, tail), previous_numerator,
        current_numerator, previous_denominator, current_denominator).first =
        continued_fraction_convergent_state(tail,
            current_numerator,
            current_numerator * head + previous_numerator,
            current_denominator,
            current_denominator * head + previous_denominator).first
}

/// The second projection of a nonempty state follows the denominator recurrence.
theorem continued_fraction_convergent_state_cons_second(head: Nat, tail: List[Nat],
    previous_numerator: Nat, current_numerator: Nat,
    previous_denominator: Nat, current_denominator: Nat) {
    continued_fraction_convergent_state(List.cons(head, tail), previous_numerator,
        current_numerator, previous_denominator, current_denominator).second =
        continued_fraction_convergent_state(tail,
            current_numerator,
            current_numerator * head + previous_numerator,
            current_denominator,
            current_denominator * head + previous_denominator).second
}

/// The singleton convergent has numerator equal to its coefficient.
theorem continued_fraction_convergent_singleton_first(head: Nat) {
    continued_fraction_convergent(List.cons(head, List.nil[Nat])).first = head
}

/// The singleton convergent has denominator one.
theorem continued_fraction_convergent_singleton_second(head: Nat) {
    continued_fraction_convergent(List.cons(head, List.nil[Nat])).second = Nat.1
}

/// A singleton continued fraction has denominator one.
theorem continued_fraction_denominator_singleton(head: Nat) {
    continued_fraction_denominator(List.cons(head, List.nil[Nat])) = Nat.1
}

/// A singleton continued fraction has denominator one.
theorem continued_fraction_singleton_denominator(head: Nat) {
    ContinuedFraction.singleton(head).denominator = Nat.1
}

/// The singleton convergent first projection matches the numerator definition.
theorem continued_fraction_convergent_singleton_first_eq_numerator(head: Nat) {
    continued_fraction_convergent(List.cons(head, List.nil[Nat])).first =
        continued_fraction_numerator(List.cons(head, List.nil[Nat]))
}

/// The singleton convergent second projection matches the denominator definition.
theorem continued_fraction_convergent_singleton_second_eq_denominator(head: Nat) {
    continued_fraction_convergent(List.cons(head, List.nil[Nat])).second =
        continued_fraction_denominator(List.cons(head, List.nil[Nat]))
}

/// The singleton convergent has positive denominator.
theorem continued_fraction_convergent_singleton_second_positive(head: Nat) {
    Nat.0 < continued_fraction_convergent(List.cons(head, List.nil[Nat])).second
}

/// The singleton convergent has nonzero denominator.
theorem continued_fraction_convergent_singleton_second_ne_zero(head: Nat) {
    continued_fraction_convergent(List.cons(head, List.nil[Nat])).second != Nat.0
}

/// A singleton continued fraction has convergent numerator equal to its coefficient.
theorem continued_fraction_singleton_convergent_first(head: Nat) {
    ContinuedFraction.singleton(head).convergent.first = head
}

/// A singleton continued fraction has convergent denominator one.
theorem continued_fraction_singleton_convergent_second(head: Nat) {
    ContinuedFraction.singleton(head).convergent.second = Nat.1
}

/// A two-term continued fraction has denominator equal to its tail coefficient.
theorem continued_fraction_denominator_pair(head: Nat, next: Nat) {
    continued_fraction_denominator(List.cons(head, List.cons(next, List.nil[Nat]))) =
        next
}

/// The two-term convergent has numerator `head * next + 1`.
theorem continued_fraction_convergent_pair_first(head: Nat, next: Nat) {
    continued_fraction_convergent(List.cons(head, List.cons(next, List.nil[Nat]))).first =
        head * next + Nat.1
}

/// The two-term convergent has denominator equal to the second coefficient.
theorem continued_fraction_convergent_pair_second(head: Nat, next: Nat) {
    continued_fraction_convergent(List.cons(head, List.cons(next, List.nil[Nat]))).second =
        next
}

/// The two-term convergent first projection matches the numerator definition.
theorem continued_fraction_convergent_pair_first_eq_numerator(head: Nat, next: Nat) {
    continued_fraction_convergent(List.cons(head, List.cons(next, List.nil[Nat]))).first =
        continued_fraction_numerator(List.cons(head, List.cons(next, List.nil[Nat])))
}

/// The two-term convergent second projection matches the denominator definition.
theorem continued_fraction_convergent_pair_second_eq_denominator(head: Nat, next: Nat) {
    continued_fraction_convergent(List.cons(head, List.cons(next, List.nil[Nat]))).second =
        continued_fraction_denominator(List.cons(head, List.cons(next, List.nil[Nat])))
}

/// A two-term convergent with positive second coefficient has positive denominator.
theorem continued_fraction_convergent_pair_second_positive_of_tail_positive(head: Nat,
    next: Nat) {
    Nat.0 < next implies
        Nat.0 < continued_fraction_convergent(
            List.cons(head, List.cons(next, List.nil[Nat]))).second
}

/// A valid two-term convergent has positive denominator.
theorem continued_fraction_convergent_pair_second_positive(head: Nat, next: Nat) {
    finite_continued_fraction_coefficients(List.cons(head, List.cons(next, List.nil[Nat])))
        implies Nat.0 < continued_fraction_convergent(
            List.cons(head, List.cons(next, List.nil[Nat]))).second
}

/// A valid two-term convergent has nonzero denominator.
theorem continued_fraction_convergent_pair_second_ne_zero(head: Nat, next: Nat) {
    finite_continued_fraction_coefficients(List.cons(head, List.cons(next, List.nil[Nat])))
        implies continued_fraction_convergent(
            List.cons(head, List.cons(next, List.nil[Nat]))).second != Nat.0
}

/// A three-term continued fraction has denominator equal to the tail continuant.
theorem continued_fraction_denominator_triple(head: Nat, next: Nat, third: Nat) {
    continued_fraction_denominator(
        List.cons(head, List.cons(next, List.cons(third, List.nil[Nat])))) =
        next * third + Nat.1
}

/// A singleton continued fraction has positive denominator.
theorem continued_fraction_denominator_singleton_positive(head: Nat) {
    Nat.0 < continued_fraction_denominator(List.cons(head, List.nil[Nat]))
}

/// A valid two-term continued fraction has positive denominator.
theorem continued_fraction_denominator_pair_positive(head: Nat, next: Nat) {
    finite_continued_fraction_coefficients(List.cons(head, List.cons(next, List.nil[Nat])))
        implies Nat.0 < continued_fraction_denominator(List.cons(head, List.cons(next, List.nil[Nat])))
}

/// A valid two-term continued fraction has nonzero denominator.
theorem continued_fraction_denominator_pair_ne_zero(head: Nat, next: Nat) {
    finite_continued_fraction_coefficients(List.cons(head, List.cons(next, List.nil[Nat])))
        implies continued_fraction_denominator(List.cons(head, List.cons(next, List.nil[Nat]))) != Nat.0
}

/// A two-term continued fraction has denominator equal to the second coefficient.
theorem continued_fraction_pair_denominator(cf: ContinuedFraction, head: Nat, next: Nat) {
    cf.coefficients = List.cons(head, List.cons(next, List.nil[Nat]))
        implies cf.denominator = next
}

/// A two-term continued fraction has positive denominator.
theorem continued_fraction_pair_denominator_positive(cf: ContinuedFraction,
    head: Nat, next: Nat) {
    cf.coefficients = List.cons(head, List.cons(next, List.nil[Nat]))
        implies Nat.0 < cf.denominator
}

/// A three-term continued fraction with positive tail coefficients has positive denominator.
theorem continued_fraction_denominator_triple_positive_of_tail_positive(head: Nat,
    next: Nat, third: Nat) {
    Nat.0 < next and Nat.0 < third implies
        Nat.0 < continued_fraction_denominator(
            List.cons(head, List.cons(next, List.cons(third, List.nil[Nat]))))
}

/// A valid three-term continued fraction has positive denominator.
theorem continued_fraction_denominator_triple_positive(head: Nat, next: Nat,
    third: Nat) {
    finite_continued_fraction_coefficients(
        List.cons(head, List.cons(next, List.cons(third, List.nil[Nat]))))
        implies Nat.0 < continued_fraction_denominator(
            List.cons(head, List.cons(next, List.cons(third, List.nil[Nat]))))
}

/// A valid three-term continued fraction has nonzero denominator.
theorem continued_fraction_denominator_triple_ne_zero(head: Nat, next: Nat,
    third: Nat) {
    finite_continued_fraction_coefficients(
        List.cons(head, List.cons(next, List.cons(third, List.nil[Nat]))))
        implies continued_fraction_denominator(
            List.cons(head, List.cons(next, List.cons(third, List.nil[Nat])))) != Nat.0
}

/// A three-term continued fraction has denominator equal to the tail continuant.
theorem continued_fraction_triple_denominator(cf: ContinuedFraction, head: Nat,
    next: Nat, third: Nat) {
    cf.coefficients = List.cons(head, List.cons(next, List.cons(third, List.nil[Nat])))
        implies cf.denominator = next * third + Nat.1
}

/// A three-term continued fraction has positive denominator.
theorem continued_fraction_triple_denominator_positive(cf: ContinuedFraction,
    head: Nat, next: Nat, third: Nat) {
    cf.coefficients = List.cons(head, List.cons(next, List.cons(third, List.nil[Nat])))
        implies Nat.0 < cf.denominator
}

/// The singleton numerator is the coefficient times the denominator.
theorem continued_fraction_numerator_singleton_recurrence(head: Nat) {
    continued_fraction_numerator(List.cons(head, List.nil[Nat])) =
        head * continued_fraction_denominator(List.cons(head, List.nil[Nat]))
}

/// The two-term numerator satisfies the first nontrivial continuant recurrence.
theorem continued_fraction_numerator_pair_recurrence(head: Nat, next: Nat) {
    continued_fraction_numerator(List.cons(head, List.cons(next, List.nil[Nat]))) =
        head * continued_fraction_denominator(List.cons(head, List.cons(next, List.nil[Nat]))) +
            Nat.1
}

/// The value of a two-term continued fraction is `head + 1 / next`.
theorem continued_fraction_value_pair(head: Nat, next: Nat) {
    continued_fraction_value(List.cons(head, List.cons(next, List.nil[Nat]))) =
        Rat.from_nat(head) + Rat.from_nat(next).inverse
}

/// The value of a two-term continued fraction is `head + 1 / next`.
theorem continued_fraction_pair_value(cf: ContinuedFraction, head: Nat, next: Nat) {
    cf.coefficients = List.cons(head, List.cons(next, List.nil[Nat]))
        implies cf.value = Rat.from_nat(head) + Rat.from_nat(next).inverse
}

/// The value of a three-term continued fraction unfolds through the tail pair.
theorem continued_fraction_value_triple(head: Nat, next: Nat, third: Nat) {
    continued_fraction_value(List.cons(head, List.cons(next, List.cons(third, List.nil[Nat])))) =
        Rat.from_nat(head) +
            (Rat.from_nat(next) + Rat.from_nat(third).inverse).inverse
}

/// The value of a three-term continued fraction unfolds through the tail pair.
theorem continued_fraction_triple_value(cf: ContinuedFraction, head: Nat, next: Nat,
    third: Nat) {
    cf.coefficients = List.cons(head, List.cons(next, List.cons(third, List.nil[Nat])))
        implies cf.value =
            Rat.from_nat(head) +
                (Rat.from_nat(next) + Rat.from_nat(third).inverse).inverse
}

/// Adding two singleton continued-fraction values adds their coefficients.
theorem continued_fraction_value_singleton_add(left: Nat, right: Nat) {
    continued_fraction_value(List.cons(left, List.nil[Nat])) +
        continued_fraction_value(List.cons(right, List.nil[Nat])) =
        Rat.from_nat(left + right)
}

/// Adding two singleton continued-fraction values adds their coefficients.
theorem continued_fraction_singleton_value_add(left: Nat, right: Nat) {
    ContinuedFraction.singleton(left).value + ContinuedFraction.singleton(right).value =
        Rat.from_nat(left + right)
}

theorem coprime_one_right(a: Nat) {
    a.coprime(Nat.1)
}

theorem coprime_comm(a: Nat, b: Nat) {
    a.coprime(b) implies b.coprime(a)
}

theorem coprime_mul(a: Nat, b: Nat, c: Nat) {
    a.coprime(b) and a.coprime(c) implies a.coprime(b * c)
}

theorem prime_divides_mul(p: Nat, a: Nat, b: Nat) {
    p.is_prime and p.divides(a * b) implies p.divides(a) or p.divides(b)
}

theorem falling_product_eq_binom_mul_factorial(n: Nat, k: Nat) {
    k < n implies falling_product(n, k) = n.binom(k.suc) * k.suc.factorial
}

theorem nat_congr_mod_iff_int_mod_rel(a: Nat, b: Nat, n: Nat) {
    a.congr_mod(b, n) = int_mod_rel(n, Int.from_nat(a), Int.from_nat(b))
}

theorem pairwise_coprime_cons_imp(head: Nat, tail: List[Nat]) {
    pairwise_coprime(List.cons(head, tail)) implies
        coprime_with_all(head, tail) and pairwise_coprime(tail)
}

theorem coprime_with_all_imp_coprime_product(a: Nat, list: List[Nat]) {
    coprime_with_all(a, list) implies a.coprime(product[Nat](list))
}

theorem coprime_residues_all_coprime(n: Nat, x: Nat) {
    coprime_residues(n).contains(x) implies x.coprime(n)
}

theorem coprime_residues_all_below(n: Nat, x: Nat) {
    coprime_residues(n).contains(x) implies x < n
}

theorem coprime_residues_contains_imp(n: Nat, x: Nat) {
    coprime_residues(n).contains(x) implies x < n and x.coprime(n)
}

theorem coprime_residues_contains_intro(n: Nat, x: Nat) {
    x < n and x.coprime(n) implies coprime_residues(n).contains(x)
}

theorem coprime_residues_unique(n: Nat) {
    coprime_residues(n).is_unique
}

// mersenne_perfect.ac
/// True if `n` is a perfect number: the sum of its positive divisors is `2 n`.
define is_perfect(n: Nat) -> Bool {
    nat_sigma(n) = Nat.2 * n
}

/// Euclid's construction: if `2^p - 1` is prime, then `2^(p-1) (2^p - 1)` is
/// perfect (Euclid's Elements, Proposition IX.36).
theorem euclid_construction(p: Nat) {
    (Nat.2.pow(p) - Nat.1).is_prime implies
        is_perfect(Nat.2.pow(p - Nat.1) * (Nat.2.pow(p) - Nat.1))
}
