from number_theory.primitive_root import Nat, is_power_of_mod, powers_cover_units_mod,
    power_of_mod_congr_mod, power_of_mod_pow, power_of_mod_mul, power_of_mod_power,
    powers_cover_units_mod_apply, powers_cover_units_mod_mul, powers_cover_units_mod_power
from number_theory.multiplicative_order import is_multiplicative_order_mod,
    multiplicative_order_positive, pow_congr_mod_multiplicative_order,
    powers_below_order_congr_imp_eq
from number_theory.congruence import congr_mod_refl, congr_mod_symm, congr_mod_trans,
    congr_mod_mul, congr_mod_pow, mod_lt
from nat import exp_add, exp_mul
numerals Nat

/// A reduced power representative for `a` modulo `n` with respect to the
/// explicit multiplicative order `ord` of the base `g`.
define is_reduced_power_of_mod(a: Nat, g: Nat, n: Nat, k: Nat, ord: Nat) -> Bool {
    k < ord and a.congr_mod(g.pow(k), n)
}

/// Projection: a reduced representative has exponent below the order.
theorem reduced_power_index_lt(a: Nat, g: Nat, n: Nat, k: Nat, ord: Nat) {
    is_reduced_power_of_mod(a, g, n, k, ord) implies k < ord
} by {
    if is_reduced_power_of_mod(a, g, n, k, ord) {
        is_reduced_power_of_mod(a, g, n, k, ord) =
            (k < ord and a.congr_mod(g.pow(k), n))
        k < ord
    }
}

/// Projection: a reduced representative represents the residue class.
theorem reduced_power_congr(a: Nat, g: Nat, n: Nat, k: Nat, ord: Nat) {
    is_reduced_power_of_mod(a, g, n, k, ord) implies a.congr_mod(g.pow(k), n)
} by {
    if is_reduced_power_of_mod(a, g, n, k, ord) {
        is_reduced_power_of_mod(a, g, n, k, ord) =
            (k < ord and a.congr_mod(g.pow(k), n))
        a.congr_mod(g.pow(k), n)
    }
}

/// If `a` is represented by `g^k`, reducing `k` modulo an explicit order of
/// `g` preserves the represented residue class.
theorem power_congr_reduced_mod_order(g: Nat, n: Nat, ord: Nat, a: Nat, k: Nat) {
    is_multiplicative_order_mod(g, n, ord) and a.congr_mod(g.pow(k), n)
        implies a.congr_mod(g.pow(k.mod(ord)), n)
} by {
    if is_multiplicative_order_mod(g, n, ord) and a.congr_mod(g.pow(k), n) {
        pow_congr_mod_multiplicative_order(g, n, ord, k)
        g.pow(k).congr_mod(g.pow(k.mod(ord)), n)
        congr_mod_trans(a, g.pow(k), g.pow(k.mod(ord)), n)
        a.congr_mod(g.pow(k.mod(ord)), n)
    }
}

/// The remainder of an exponent modulo an explicit multiplicative order is
/// below that order.
theorem power_reduced_index_lt_order(g: Nat, n: Nat, ord: Nat, k: Nat) {
    is_multiplicative_order_mod(g, n, ord) implies k.mod(ord) < ord
} by {
    if is_multiplicative_order_mod(g, n, ord) {
        multiplicative_order_positive(g, n, ord)
        Nat.0 < ord
        ord != Nat.0
        mod_lt(k, ord)
        k.mod(ord) < ord
    }
}

/// A concrete power representative can always be replaced by a representative
/// whose exponent is reduced modulo the explicit order.
theorem reduced_power_of_power_mod_order(g: Nat, n: Nat, ord: Nat, a: Nat, k: Nat) {
    is_multiplicative_order_mod(g, n, ord) and a.congr_mod(g.pow(k), n)
        implies is_reduced_power_of_mod(a, g, n, k.mod(ord), ord)
} by {
    if is_multiplicative_order_mod(g, n, ord) and a.congr_mod(g.pow(k), n) {
        power_reduced_index_lt_order(g, n, ord, k)
        k.mod(ord) < ord
        power_congr_reduced_mod_order(g, n, ord, a, k)
        a.congr_mod(g.pow(k.mod(ord)), n)
        is_reduced_power_of_mod(a, g, n, k.mod(ord), ord) =
            (k.mod(ord) < ord and a.congr_mod(g.pow(k.mod(ord)), n))
        is_reduced_power_of_mod(a, g, n, k.mod(ord), ord)
    }
}

/// A below-order exponent representing a residue is a reduced representative.
theorem reduced_power_of_below_order_congr(a: Nat, g: Nat, n: Nat, k: Nat, ord: Nat) {
    k < ord and a.congr_mod(g.pow(k), n)
        implies is_reduced_power_of_mod(a, g, n, k, ord)
} by {
    if k < ord and a.congr_mod(g.pow(k), n) {
        is_reduced_power_of_mod(a, g, n, k, ord) =
            (k < ord and a.congr_mod(g.pow(k), n))
        is_reduced_power_of_mod(a, g, n, k, ord)
    }
}

/// Any power representation admits a reduced representative for an explicit
/// multiplicative order.
theorem reduced_power_exists_of_power(g: Nat, n: Nat, ord: Nat, a: Nat) {
    is_multiplicative_order_mod(g, n, ord) and is_power_of_mod(a, g, n)
        implies exists(k: Nat) { is_reduced_power_of_mod(a, g, n, k, ord) }
} by {
    if is_multiplicative_order_mod(g, n, ord) and is_power_of_mod(a, g, n) {
        is_power_of_mod(a, g, n) = exists(k: Nat) { a.congr_mod(g.pow(k), n) }
        let k: Nat satisfy { a.congr_mod(g.pow(k), n) }
        reduced_power_of_power_mod_order(g, n, ord, a, k)
        is_reduced_power_of_mod(a, g, n, k.mod(ord), ord)
        exists(j: Nat) { is_reduced_power_of_mod(a, g, n, j, ord) }
    }
}

/// Unit coverage by powers supplies a reduced representative for every unit.
theorem reduced_power_exists_of_unit_coverage(g: Nat, n: Nat, ord: Nat, a: Nat) {
    is_multiplicative_order_mod(g, n, ord) and powers_cover_units_mod(g, n)
        and a.coprime(n)
        implies exists(k: Nat) { is_reduced_power_of_mod(a, g, n, k, ord) }
} by {
    if is_multiplicative_order_mod(g, n, ord) and powers_cover_units_mod(g, n)
        and a.coprime(n) {
        powers_cover_units_mod_apply(g, n, a)
        is_power_of_mod(a, g, n)
        reduced_power_exists_of_power(g, n, ord, a)
        exists(k: Nat) { is_reduced_power_of_mod(a, g, n, k, ord) }
    }
}

/// If two elements have the same representative `a`, their represented powers
/// are congruent to each other.
theorem powers_congr_of_common_representative(g: Nat, n: Nat, a: Nat, i: Nat, j: Nat) {
    a.congr_mod(g.pow(i), n) and a.congr_mod(g.pow(j), n)
        implies g.pow(i).congr_mod(g.pow(j), n)
} by {
    if a.congr_mod(g.pow(i), n) and a.congr_mod(g.pow(j), n) {
        a.congr_mod(g.pow(i), n) = (a.mod(n) = g.pow(i).mod(n))
        a.congr_mod(g.pow(j), n) = (a.mod(n) = g.pow(j).mod(n))
        g.pow(i).mod(n) = a.mod(n)
        g.pow(i).mod(n) = g.pow(j).mod(n)
        g.pow(i).congr_mod(g.pow(j), n)
    }
}

/// Predicate form of uniqueness for reduced representatives.
theorem reduced_power_representatives_unique(
    g: Nat, n: Nat, ord: Nat, a: Nat, i: Nat, j: Nat
) {
    is_multiplicative_order_mod(g, n, ord)
        and is_reduced_power_of_mod(a, g, n, i, ord)
        and is_reduced_power_of_mod(a, g, n, j, ord)
        implies i = j
} by {
    if is_multiplicative_order_mod(g, n, ord)
        and is_reduced_power_of_mod(a, g, n, i, ord)
        and is_reduced_power_of_mod(a, g, n, j, ord) {
        reduced_power_index_lt(a, g, n, i, ord)
        i < ord
        reduced_power_congr(a, g, n, i, ord)
        a.congr_mod(g.pow(i), n)
        reduced_power_index_lt(a, g, n, j, ord)
        j < ord
        reduced_power_congr(a, g, n, j, ord)
        a.congr_mod(g.pow(j), n)
        powers_congr_of_common_representative(g, n, a, i, j)
        g.pow(i).congr_mod(g.pow(j), n)
        powers_below_order_congr_imp_eq(g, n, ord, i, j)
        i = j
    }
}

/// A canonical reduced exponent for a represented residue class with respect
/// to an explicit order. Outside the order-and-representation domain it falls
/// back to `0`.
let discrete_log_mod_with_order(g: Nat, n: Nat, ord: Nat, a: Nat) -> k: Nat satisfy {
    (is_multiplicative_order_mod(g, n, ord) and is_power_of_mod(a, g, n)
        and is_reduced_power_of_mod(a, g, n, k, ord))
    or ((not is_multiplicative_order_mod(g, n, ord) or not is_power_of_mod(a, g, n))
        and k = Nat.0)
} by {
    if is_multiplicative_order_mod(g, n, ord) and is_power_of_mod(a, g, n) {
        reduced_power_exists_of_power(g, n, ord, a)
        let k: Nat satisfy { is_reduced_power_of_mod(a, g, n, k, ord) }
        is_multiplicative_order_mod(g, n, ord) and is_power_of_mod(a, g, n) and is_reduced_power_of_mod(a, g, n, k, ord)
        (is_multiplicative_order_mod(g, n, ord) and is_power_of_mod(a, g, n) and is_reduced_power_of_mod(a, g, n, k, ord)) or ((not is_multiplicative_order_mod(g, n, ord) or not is_power_of_mod(a, g, n)) and k = Nat.0)
    } else {
        Nat.0 = Nat.0
        not is_multiplicative_order_mod(g, n, ord) or not is_power_of_mod(a, g, n)
        (not is_multiplicative_order_mod(g, n, ord) or not is_power_of_mod(a, g, n)) and Nat.0 = Nat.0
        (is_multiplicative_order_mod(g, n, ord) and is_power_of_mod(a, g, n) and is_reduced_power_of_mod(a, g, n, Nat.0, ord)) or ((not is_multiplicative_order_mod(g, n, ord) or not is_power_of_mod(a, g, n)) and Nat.0 = Nat.0)
    }
}

/// Total specification of the chosen explicit-order discrete logarithm.
theorem discrete_log_mod_with_order_total_spec(g: Nat, n: Nat, ord: Nat, a: Nat) {
    (is_multiplicative_order_mod(g, n, ord) and is_power_of_mod(a, g, n)
        and is_reduced_power_of_mod(a, g, n, discrete_log_mod_with_order(g, n, ord, a), ord))
    or ((not is_multiplicative_order_mod(g, n, ord) or not is_power_of_mod(a, g, n))
        and discrete_log_mod_with_order(g, n, ord, a) = Nat.0)
}

/// The expected reduced exponent for the product of two represented residues.
define discrete_log_mod_with_order_mul_index(
    g: Nat, n: Nat, ord: Nat, a: Nat, b: Nat
) -> Nat {
    (discrete_log_mod_with_order(g, n, ord, a) +
        discrete_log_mod_with_order(g, n, ord, b)).mod(ord)
}

/// The expected reduced exponent for a power of a represented residue.
define discrete_log_mod_with_order_power_index(
    g: Nat, n: Nat, ord: Nat, a: Nat, m: Nat
) -> Nat {
    (discrete_log_mod_with_order(g, n, ord, a) * m).mod(ord)
}

/// On the represented domain, the chosen explicit-order discrete logarithm is
/// a reduced representative.
theorem discrete_log_mod_with_order_is_reduced(g: Nat, n: Nat, ord: Nat, a: Nat) {
    is_multiplicative_order_mod(g, n, ord) and is_power_of_mod(a, g, n)
        implies is_reduced_power_of_mod(a, g, n, discrete_log_mod_with_order(g, n, ord, a), ord)
} by {
    if is_multiplicative_order_mod(g, n, ord) and is_power_of_mod(a, g, n) {
        discrete_log_mod_with_order_total_spec(g, n, ord, a)
        if is_multiplicative_order_mod(g, n, ord) and is_power_of_mod(a, g, n)
            and is_reduced_power_of_mod(a, g, n, discrete_log_mod_with_order(g, n, ord, a), ord) {
        } else {
            (not is_multiplicative_order_mod(g, n, ord) or not is_power_of_mod(a, g, n)) and discrete_log_mod_with_order(g, n, ord, a) = Nat.0
            false
        }
    }
}

/// The chosen explicit-order discrete logarithm is below the order.
theorem discrete_log_mod_with_order_lt_order(g: Nat, n: Nat, ord: Nat, a: Nat) {
    is_multiplicative_order_mod(g, n, ord) and is_power_of_mod(a, g, n)
        implies discrete_log_mod_with_order(g, n, ord, a) < ord
} by {
    if is_multiplicative_order_mod(g, n, ord) and is_power_of_mod(a, g, n) {
        discrete_log_mod_with_order_is_reduced(g, n, ord, a)
        reduced_power_index_lt(a, g, n, discrete_log_mod_with_order(g, n, ord, a), ord)
        discrete_log_mod_with_order(g, n, ord, a) < ord
    }
}

/// The chosen explicit-order discrete logarithm represents the original residue.
theorem discrete_log_mod_with_order_congr(g: Nat, n: Nat, ord: Nat, a: Nat) {
    is_multiplicative_order_mod(g, n, ord) and is_power_of_mod(a, g, n)
        implies a.congr_mod(g.pow(discrete_log_mod_with_order(g, n, ord, a)), n)
} by {
    if is_multiplicative_order_mod(g, n, ord) and is_power_of_mod(a, g, n) {
        discrete_log_mod_with_order_is_reduced(g, n, ord, a)
        reduced_power_congr(a, g, n, discrete_log_mod_with_order(g, n, ord, a), ord)
        a.congr_mod(g.pow(discrete_log_mod_with_order(g, n, ord, a)), n)
    }
}

/// Under unit coverage, the chosen explicit-order discrete logarithm is reduced.
theorem unit_discrete_log_mod_with_order_is_reduced(g: Nat, n: Nat, ord: Nat, a: Nat) {
    is_multiplicative_order_mod(g, n, ord) and powers_cover_units_mod(g, n)
        and a.coprime(n)
        implies is_reduced_power_of_mod(a, g, n, discrete_log_mod_with_order(g, n, ord, a), ord)
} by {
    if is_multiplicative_order_mod(g, n, ord) and powers_cover_units_mod(g, n)
        and a.coprime(n) {
        powers_cover_units_mod_apply(g, n, a)
        is_power_of_mod(a, g, n)
        discrete_log_mod_with_order_is_reduced(g, n, ord, a)
        is_reduced_power_of_mod(a, g, n, discrete_log_mod_with_order(g, n, ord, a), ord)
    }
}

/// Under unit coverage, the chosen explicit-order discrete logarithm is below the order.
theorem unit_discrete_log_mod_with_order_lt_order(g: Nat, n: Nat, ord: Nat, a: Nat) {
    is_multiplicative_order_mod(g, n, ord) and powers_cover_units_mod(g, n)
        and a.coprime(n)
        implies discrete_log_mod_with_order(g, n, ord, a) < ord
} by {
    if is_multiplicative_order_mod(g, n, ord) and powers_cover_units_mod(g, n)
        and a.coprime(n) {
        unit_discrete_log_mod_with_order_is_reduced(g, n, ord, a)
        reduced_power_index_lt(a, g, n, discrete_log_mod_with_order(g, n, ord, a), ord)
        discrete_log_mod_with_order(g, n, ord, a) < ord
    }
}

/// Under unit coverage, the chosen explicit-order discrete logarithm represents the unit.
theorem unit_discrete_log_mod_with_order_congr(g: Nat, n: Nat, ord: Nat, a: Nat) {
    is_multiplicative_order_mod(g, n, ord) and powers_cover_units_mod(g, n)
        and a.coprime(n)
        implies a.congr_mod(g.pow(discrete_log_mod_with_order(g, n, ord, a)), n)
} by {
    if is_multiplicative_order_mod(g, n, ord) and powers_cover_units_mod(g, n)
        and a.coprime(n) {
        unit_discrete_log_mod_with_order_is_reduced(g, n, ord, a)
        reduced_power_congr(a, g, n, discrete_log_mod_with_order(g, n, ord, a), ord)
        a.congr_mod(g.pow(discrete_log_mod_with_order(g, n, ord, a)), n)
    }
}

/// The chosen explicit-order discrete logarithm is unique among all reduced
/// representatives.
theorem discrete_log_mod_with_order_unique(g: Nat, n: Nat, ord: Nat, a: Nat, k: Nat) {
    is_multiplicative_order_mod(g, n, ord) and is_power_of_mod(a, g, n)
        and is_reduced_power_of_mod(a, g, n, k, ord)
        implies k = discrete_log_mod_with_order(g, n, ord, a)
} by {
    if is_multiplicative_order_mod(g, n, ord) and is_power_of_mod(a, g, n)
        and is_reduced_power_of_mod(a, g, n, k, ord) {
        discrete_log_mod_with_order_is_reduced(g, n, ord, a)
        is_reduced_power_of_mod(a, g, n, discrete_log_mod_with_order(g, n, ord, a), ord)
        reduced_power_representatives_unique(
            g, n, ord, a, k, discrete_log_mod_with_order(g, n, ord, a))
        k = discrete_log_mod_with_order(g, n, ord, a)
    }
}

/// The chosen explicit-order discrete logarithm also represents the original
/// residue with the congruence reversed.
theorem discrete_log_mod_with_order_pow_congr(g: Nat, n: Nat, ord: Nat, a: Nat) {
    is_multiplicative_order_mod(g, n, ord) and is_power_of_mod(a, g, n)
        implies g.pow(discrete_log_mod_with_order(g, n, ord, a)).congr_mod(a, n)
} by {
    if is_multiplicative_order_mod(g, n, ord) and is_power_of_mod(a, g, n) {
        discrete_log_mod_with_order_congr(g, n, ord, a)
        a.congr_mod(g.pow(discrete_log_mod_with_order(g, n, ord, a)), n)
        congr_mod_symm(a, g.pow(discrete_log_mod_with_order(g, n, ord, a)), n)
        g.pow(discrete_log_mod_with_order(g, n, ord, a)).congr_mod(a, n)
    }
}

/// Under unit coverage, the chosen explicit-order discrete logarithm also
/// represents the unit with the congruence reversed.
theorem unit_discrete_log_mod_with_order_pow_congr(g: Nat, n: Nat, ord: Nat, a: Nat) {
    is_multiplicative_order_mod(g, n, ord) and powers_cover_units_mod(g, n)
        and a.coprime(n)
        implies g.pow(discrete_log_mod_with_order(g, n, ord, a)).congr_mod(a, n)
} by {
    if is_multiplicative_order_mod(g, n, ord) and powers_cover_units_mod(g, n)
        and a.coprime(n) {
        unit_discrete_log_mod_with_order_congr(g, n, ord, a)
        a.congr_mod(g.pow(discrete_log_mod_with_order(g, n, ord, a)), n)
        congr_mod_symm(a, g.pow(discrete_log_mod_with_order(g, n, ord, a)), n)
        g.pow(discrete_log_mod_with_order(g, n, ord, a)).congr_mod(a, n)
    }
}

/// A reduced representative is the chosen explicit-order discrete logarithm.
theorem discrete_log_mod_with_order_of_reduced_power(
    g: Nat, n: Nat, ord: Nat, a: Nat, k: Nat
) {
    is_multiplicative_order_mod(g, n, ord) and is_power_of_mod(a, g, n)
        and is_reduced_power_of_mod(a, g, n, k, ord)
        implies discrete_log_mod_with_order(g, n, ord, a) = k
} by {
    if is_multiplicative_order_mod(g, n, ord) and is_power_of_mod(a, g, n)
        and is_reduced_power_of_mod(a, g, n, k, ord) {
        discrete_log_mod_with_order_unique(g, n, ord, a, k)
        k = discrete_log_mod_with_order(g, n, ord, a)
        discrete_log_mod_with_order(g, n, ord, a) = k
    }
}

/// The discrete logarithm of a literal power is the exponent reduced modulo
/// the order.
theorem discrete_log_mod_with_order_of_power(g: Nat, n: Nat, ord: Nat, k: Nat) {
    is_multiplicative_order_mod(g, n, ord)
        implies discrete_log_mod_with_order(g, n, ord, g.pow(k)) = k.mod(ord)
} by {
    if is_multiplicative_order_mod(g, n, ord) {
        power_of_mod_pow(g, k, n)
        is_power_of_mod(g.pow(k), g, n)
        congr_mod_refl(g.pow(k), n)
        g.pow(k).congr_mod(g.pow(k), n)
        reduced_power_of_power_mod_order(g, n, ord, g.pow(k), k)
        is_reduced_power_of_mod(g.pow(k), g, n, k.mod(ord), ord)
        discrete_log_mod_with_order_of_reduced_power(
            g, n, ord, g.pow(k), k.mod(ord))
        discrete_log_mod_with_order(g, n, ord, g.pow(k)) = k.mod(ord)
    }
}

/// The product of two represented residues has the reduced representative given
/// by the sum of their chosen discrete logarithms.
theorem reduced_power_of_discrete_log_mul(
    g: Nat, n: Nat, ord: Nat, a: Nat, b: Nat
) {
    is_multiplicative_order_mod(g, n, ord)
        and is_power_of_mod(a, g, n) and is_power_of_mod(b, g, n)
        implies is_reduced_power_of_mod(a * b, g, n,
            discrete_log_mod_with_order_mul_index(g, n, ord, a, b), ord)
} by {
    if is_multiplicative_order_mod(g, n, ord)
        and is_power_of_mod(a, g, n) and is_power_of_mod(b, g, n) {
        discrete_log_mod_with_order_congr(g, n, ord, a)
        a.congr_mod(g.pow(discrete_log_mod_with_order(g, n, ord, a)), n)
        discrete_log_mod_with_order_congr(g, n, ord, b)
        b.congr_mod(g.pow(discrete_log_mod_with_order(g, n, ord, b)), n)
        congr_mod_mul(
            a, b,
            g.pow(discrete_log_mod_with_order(g, n, ord, a)),
            g.pow(discrete_log_mod_with_order(g, n, ord, b)),
            n)
        (a * b).congr_mod(
            g.pow(discrete_log_mod_with_order(g, n, ord, a)) *
                g.pow(discrete_log_mod_with_order(g, n, ord, b)),
            n)
        exp_add(
            g,
            discrete_log_mod_with_order(g, n, ord, a),
            discrete_log_mod_with_order(g, n, ord, b))
        g.pow(
            discrete_log_mod_with_order(g, n, ord, a) +
                discrete_log_mod_with_order(g, n, ord, b)) =
            g.pow(discrete_log_mod_with_order(g, n, ord, a)) *
                g.pow(discrete_log_mod_with_order(g, n, ord, b))
        (a * b).congr_mod(
            g.pow(
                discrete_log_mod_with_order(g, n, ord, a) +
                    discrete_log_mod_with_order(g, n, ord, b)),
            n)
        reduced_power_of_power_mod_order(
            g, n, ord, a * b,
            discrete_log_mod_with_order(g, n, ord, a) +
                discrete_log_mod_with_order(g, n, ord, b))
        is_reduced_power_of_mod(a * b, g, n,
            (discrete_log_mod_with_order(g, n, ord, a) +
                discrete_log_mod_with_order(g, n, ord, b)).mod(ord),
            ord)
        discrete_log_mod_with_order_mul_index(g, n, ord, a, b) =
            (discrete_log_mod_with_order(g, n, ord, a) +
                discrete_log_mod_with_order(g, n, ord, b)).mod(ord)
        is_reduced_power_of_mod(a * b, g, n,
            discrete_log_mod_with_order_mul_index(g, n, ord, a, b), ord)
    }
}

/// Multiplying a represented residue by a literal generator power has the
/// product reduced representative.
theorem reduced_power_of_discrete_log_mul_generator_power(
    g: Nat, n: Nat, ord: Nat, a: Nat, k: Nat
) {
    is_multiplicative_order_mod(g, n, ord) and is_power_of_mod(a, g, n)
        implies is_reduced_power_of_mod(a * g.pow(k), g, n,
            discrete_log_mod_with_order_mul_index(g, n, ord, a, g.pow(k)), ord)
} by {
    if is_multiplicative_order_mod(g, n, ord) and is_power_of_mod(a, g, n) {
        power_of_mod_pow(g, k, n)
        is_power_of_mod(g.pow(k), g, n)
        reduced_power_of_discrete_log_mul(g, n, ord, a, g.pow(k))
        is_reduced_power_of_mod(a * g.pow(k), g, n,
            discrete_log_mod_with_order_mul_index(g, n, ord, a, g.pow(k)), ord)
    }
}

/// Multiplying a literal generator power by a represented residue has the
/// product reduced representative.
theorem reduced_power_of_discrete_log_generator_power_mul(
    g: Nat, n: Nat, ord: Nat, k: Nat, a: Nat
) {
    is_multiplicative_order_mod(g, n, ord) and is_power_of_mod(a, g, n)
        implies is_reduced_power_of_mod(g.pow(k) * a, g, n,
            discrete_log_mod_with_order_mul_index(g, n, ord, g.pow(k), a), ord)
} by {
    if is_multiplicative_order_mod(g, n, ord) and is_power_of_mod(a, g, n) {
        power_of_mod_pow(g, k, n)
        is_power_of_mod(g.pow(k), g, n)
        reduced_power_of_discrete_log_mul(g, n, ord, g.pow(k), a)
        is_reduced_power_of_mod(g.pow(k) * a, g, n,
            discrete_log_mod_with_order_mul_index(g, n, ord, g.pow(k), a), ord)
    }
}

/// Under unit coverage, multiplying a unit by a literal generator power has
/// the product reduced representative.
theorem unit_reduced_power_of_discrete_log_mul_generator_power(
    g: Nat, n: Nat, ord: Nat, a: Nat, k: Nat
) {
    is_multiplicative_order_mod(g, n, ord) and powers_cover_units_mod(g, n)
        and a.coprime(n)
        implies is_reduced_power_of_mod(a * g.pow(k), g, n,
            discrete_log_mod_with_order_mul_index(g, n, ord, a, g.pow(k)), ord)
} by {
    if is_multiplicative_order_mod(g, n, ord) and powers_cover_units_mod(g, n)
        and a.coprime(n) {
        powers_cover_units_mod_apply(g, n, a)
        is_power_of_mod(a, g, n)
        reduced_power_of_discrete_log_mul_generator_power(g, n, ord, a, k)
        is_reduced_power_of_mod(a * g.pow(k), g, n,
            discrete_log_mod_with_order_mul_index(g, n, ord, a, g.pow(k)), ord)
    }
}

/// Under unit coverage, multiplying a literal generator power by a unit has
/// the product reduced representative.
theorem unit_reduced_power_of_discrete_log_generator_power_mul(
    g: Nat, n: Nat, ord: Nat, k: Nat, a: Nat
) {
    is_multiplicative_order_mod(g, n, ord) and powers_cover_units_mod(g, n)
        and a.coprime(n)
        implies is_reduced_power_of_mod(g.pow(k) * a, g, n,
            discrete_log_mod_with_order_mul_index(g, n, ord, g.pow(k), a), ord)
} by {
    if is_multiplicative_order_mod(g, n, ord) and powers_cover_units_mod(g, n)
        and a.coprime(n) {
        powers_cover_units_mod_apply(g, n, a)
        is_power_of_mod(a, g, n)
        reduced_power_of_discrete_log_generator_power_mul(g, n, ord, k, a)
        is_reduced_power_of_mod(g.pow(k) * a, g, n,
            discrete_log_mod_with_order_mul_index(g, n, ord, g.pow(k), a), ord)
    }
}

/// The product index unfolds to the sum of the chosen discrete logarithms,
/// reduced modulo the order.
theorem discrete_log_mod_with_order_mul_index_eq(
    g: Nat, n: Nat, ord: Nat, a: Nat, b: Nat
) {
    discrete_log_mod_with_order_mul_index(g, n, ord, a, b) =
        (discrete_log_mod_with_order(g, n, ord, a) +
            discrete_log_mod_with_order(g, n, ord, b)).mod(ord)
}

/// The discrete logarithm of a product is its product index.
theorem discrete_log_mod_with_order_mul_indexed(
    g: Nat, n: Nat, ord: Nat, a: Nat, b: Nat
) {
    is_multiplicative_order_mod(g, n, ord)
        and is_power_of_mod(a, g, n) and is_power_of_mod(b, g, n)
        implies discrete_log_mod_with_order(g, n, ord, a * b) =
            discrete_log_mod_with_order_mul_index(g, n, ord, a, b)
} by {
    if is_multiplicative_order_mod(g, n, ord)
        and is_power_of_mod(a, g, n) and is_power_of_mod(b, g, n) {
        power_of_mod_mul(a, b, g, n)
        is_power_of_mod(a * b, g, n)
        reduced_power_of_discrete_log_mul(g, n, ord, a, b)
        is_reduced_power_of_mod(a * b, g, n,
            discrete_log_mod_with_order_mul_index(g, n, ord, a, b), ord)
        discrete_log_mod_with_order_is_reduced(g, n, ord, a * b)
        is_reduced_power_of_mod(a * b, g, n,
            discrete_log_mod_with_order(g, n, ord, a * b), ord)
        reduced_power_representatives_unique(
            g, n, ord, a * b,
            discrete_log_mod_with_order_mul_index(g, n, ord, a, b),
            discrete_log_mod_with_order(g, n, ord, a * b))
        discrete_log_mod_with_order_mul_index(g, n, ord, a, b) =
            discrete_log_mod_with_order(g, n, ord, a * b)
        discrete_log_mod_with_order(g, n, ord, a * b) =
            discrete_log_mod_with_order_mul_index(g, n, ord, a, b)
    }
}

/// The discrete logarithm of a represented residue multiplied by a literal
/// generator power is its product index.
theorem discrete_log_mod_with_order_mul_generator_power_indexed(
    g: Nat, n: Nat, ord: Nat, a: Nat, k: Nat
) {
    is_multiplicative_order_mod(g, n, ord) and is_power_of_mod(a, g, n)
        implies discrete_log_mod_with_order(g, n, ord, a * g.pow(k)) =
            discrete_log_mod_with_order_mul_index(g, n, ord, a, g.pow(k))
} by {
    if is_multiplicative_order_mod(g, n, ord) and is_power_of_mod(a, g, n) {
        power_of_mod_pow(g, k, n)
        is_power_of_mod(g.pow(k), g, n)
        discrete_log_mod_with_order_mul_indexed(g, n, ord, a, g.pow(k))
        discrete_log_mod_with_order(g, n, ord, a * g.pow(k)) =
            discrete_log_mod_with_order_mul_index(g, n, ord, a, g.pow(k))
    }
}

/// The discrete logarithm of a literal generator power multiplied by a
/// represented residue is its product index.
theorem discrete_log_mod_with_order_generator_power_mul_indexed(
    g: Nat, n: Nat, ord: Nat, k: Nat, a: Nat
) {
    is_multiplicative_order_mod(g, n, ord) and is_power_of_mod(a, g, n)
        implies discrete_log_mod_with_order(g, n, ord, g.pow(k) * a) =
            discrete_log_mod_with_order_mul_index(g, n, ord, g.pow(k), a)
} by {
    if is_multiplicative_order_mod(g, n, ord) and is_power_of_mod(a, g, n) {
        power_of_mod_pow(g, k, n)
        is_power_of_mod(g.pow(k), g, n)
        discrete_log_mod_with_order_mul_indexed(g, n, ord, g.pow(k), a)
        discrete_log_mod_with_order(g, n, ord, g.pow(k) * a) =
            discrete_log_mod_with_order_mul_index(g, n, ord, g.pow(k), a)
    }
}

/// The product of two literal generator powers has the product reduced
/// representative.
theorem reduced_power_of_discrete_log_generator_power_mul_generator_power(
    g: Nat, n: Nat, ord: Nat, i: Nat, j: Nat
) {
    is_multiplicative_order_mod(g, n, ord)
        implies is_reduced_power_of_mod(g.pow(i) * g.pow(j), g, n,
            discrete_log_mod_with_order_mul_index(g, n, ord, g.pow(i), g.pow(j)), ord)
} by {
    if is_multiplicative_order_mod(g, n, ord) {
        power_of_mod_pow(g, i, n)
        is_power_of_mod(g.pow(i), g, n)
        power_of_mod_pow(g, j, n)
        is_power_of_mod(g.pow(j), g, n)
        reduced_power_of_discrete_log_mul(g, n, ord, g.pow(i), g.pow(j))
        is_reduced_power_of_mod(g.pow(i) * g.pow(j), g, n,
            discrete_log_mod_with_order_mul_index(g, n, ord, g.pow(i), g.pow(j)), ord)
    }
}

/// The discrete logarithm of a product of two literal generator powers is its
/// product index.
theorem discrete_log_mod_with_order_generator_power_mul_generator_power_indexed(
    g: Nat, n: Nat, ord: Nat, i: Nat, j: Nat
) {
    is_multiplicative_order_mod(g, n, ord)
        implies discrete_log_mod_with_order(g, n, ord, g.pow(i) * g.pow(j)) =
            discrete_log_mod_with_order_mul_index(g, n, ord, g.pow(i), g.pow(j))
} by {
    if is_multiplicative_order_mod(g, n, ord) {
        power_of_mod_pow(g, i, n)
        is_power_of_mod(g.pow(i), g, n)
        power_of_mod_pow(g, j, n)
        is_power_of_mod(g.pow(j), g, n)
        discrete_log_mod_with_order_mul_indexed(g, n, ord, g.pow(i), g.pow(j))
        discrete_log_mod_with_order(g, n, ord, g.pow(i) * g.pow(j)) =
            discrete_log_mod_with_order_mul_index(g, n, ord, g.pow(i), g.pow(j))
    }
}

/// The discrete logarithm of a product is the sum of the discrete logarithms,
/// reduced modulo the order.
theorem discrete_log_mod_with_order_mul(
    g: Nat, n: Nat, ord: Nat, a: Nat, b: Nat
) {
    is_multiplicative_order_mod(g, n, ord)
        and is_power_of_mod(a, g, n) and is_power_of_mod(b, g, n)
        implies discrete_log_mod_with_order(g, n, ord, a * b) =
            (discrete_log_mod_with_order(g, n, ord, a) +
                discrete_log_mod_with_order(g, n, ord, b)).mod(ord)
} by {
    if is_multiplicative_order_mod(g, n, ord)
        and is_power_of_mod(a, g, n) and is_power_of_mod(b, g, n) {
        discrete_log_mod_with_order_mul_indexed(g, n, ord, a, b)
        discrete_log_mod_with_order(g, n, ord, a * b) =
            discrete_log_mod_with_order_mul_index(g, n, ord, a, b)
        discrete_log_mod_with_order_mul_index_eq(g, n, ord, a, b)
        discrete_log_mod_with_order_mul_index(g, n, ord, a, b) =
            (discrete_log_mod_with_order(g, n, ord, a) +
                discrete_log_mod_with_order(g, n, ord, b)).mod(ord)
        discrete_log_mod_with_order(g, n, ord, a * b) =
            (discrete_log_mod_with_order(g, n, ord, a) +
                discrete_log_mod_with_order(g, n, ord, b)).mod(ord)
    }
}

/// A power of a represented residue has the reduced representative given by the
/// chosen discrete logarithm multiplied by the exponent.
theorem reduced_power_of_discrete_log_power(
    g: Nat, n: Nat, ord: Nat, a: Nat, m: Nat
) {
    is_multiplicative_order_mod(g, n, ord) and is_power_of_mod(a, g, n)
        implies is_reduced_power_of_mod(a.pow(m), g, n,
            discrete_log_mod_with_order_power_index(g, n, ord, a, m), ord)
} by {
    if is_multiplicative_order_mod(g, n, ord) and is_power_of_mod(a, g, n) {
        discrete_log_mod_with_order_congr(g, n, ord, a)
        a.congr_mod(g.pow(discrete_log_mod_with_order(g, n, ord, a)), n)
        congr_mod_pow(a, g.pow(discrete_log_mod_with_order(g, n, ord, a)), n, m)
        a.pow(m).congr_mod(
            g.pow(discrete_log_mod_with_order(g, n, ord, a)).pow(m), n)
        exp_mul(g, discrete_log_mod_with_order(g, n, ord, a), m)
        g.pow(discrete_log_mod_with_order(g, n, ord, a) * m) =
            g.pow(discrete_log_mod_with_order(g, n, ord, a)).pow(m)
        a.pow(m).congr_mod(
            g.pow(discrete_log_mod_with_order(g, n, ord, a) * m), n)
        reduced_power_of_power_mod_order(
            g, n, ord, a.pow(m),
            discrete_log_mod_with_order(g, n, ord, a) * m)
        is_reduced_power_of_mod(a.pow(m), g, n,
            (discrete_log_mod_with_order(g, n, ord, a) * m).mod(ord),
            ord)
        discrete_log_mod_with_order_power_index(g, n, ord, a, m) =
            (discrete_log_mod_with_order(g, n, ord, a) * m).mod(ord)
        is_reduced_power_of_mod(a.pow(m), g, n,
            discrete_log_mod_with_order_power_index(g, n, ord, a, m), ord)
    }
}

/// The power index unfolds to the chosen discrete logarithm multiplied by the
/// exponent, reduced modulo the order.
theorem discrete_log_mod_with_order_power_index_eq(
    g: Nat, n: Nat, ord: Nat, a: Nat, m: Nat
) {
    discrete_log_mod_with_order_power_index(g, n, ord, a, m) =
        (discrete_log_mod_with_order(g, n, ord, a) * m).mod(ord)
}

/// The discrete logarithm of a power is its power index.
theorem discrete_log_mod_with_order_power_indexed(
    g: Nat, n: Nat, ord: Nat, a: Nat, m: Nat
) {
    is_multiplicative_order_mod(g, n, ord) and is_power_of_mod(a, g, n)
        implies discrete_log_mod_with_order(g, n, ord, a.pow(m)) =
            discrete_log_mod_with_order_power_index(g, n, ord, a, m)
} by {
    if is_multiplicative_order_mod(g, n, ord) and is_power_of_mod(a, g, n) {
        power_of_mod_power(a, g, n, m)
        is_power_of_mod(a.pow(m), g, n)
        reduced_power_of_discrete_log_power(g, n, ord, a, m)
        is_reduced_power_of_mod(a.pow(m), g, n,
            discrete_log_mod_with_order_power_index(g, n, ord, a, m), ord)
        discrete_log_mod_with_order_is_reduced(g, n, ord, a.pow(m))
        is_reduced_power_of_mod(a.pow(m), g, n,
            discrete_log_mod_with_order(g, n, ord, a.pow(m)), ord)
        reduced_power_representatives_unique(
            g, n, ord, a.pow(m),
            discrete_log_mod_with_order_power_index(g, n, ord, a, m),
            discrete_log_mod_with_order(g, n, ord, a.pow(m)))
        discrete_log_mod_with_order_power_index(g, n, ord, a, m) =
            discrete_log_mod_with_order(g, n, ord, a.pow(m))
        discrete_log_mod_with_order(g, n, ord, a.pow(m)) =
            discrete_log_mod_with_order_power_index(g, n, ord, a, m)
    }
}

/// A power of a literal generator power has the power reduced representative.
theorem reduced_power_of_discrete_log_generator_power_power(
    g: Nat, n: Nat, ord: Nat, k: Nat, m: Nat
) {
    is_multiplicative_order_mod(g, n, ord)
        implies is_reduced_power_of_mod(g.pow(k).pow(m), g, n,
            discrete_log_mod_with_order_power_index(g, n, ord, g.pow(k), m), ord)
} by {
    if is_multiplicative_order_mod(g, n, ord) {
        power_of_mod_pow(g, k, n)
        is_power_of_mod(g.pow(k), g, n)
        reduced_power_of_discrete_log_power(g, n, ord, g.pow(k), m)
        is_reduced_power_of_mod(g.pow(k).pow(m), g, n,
            discrete_log_mod_with_order_power_index(g, n, ord, g.pow(k), m), ord)
    }
}

/// The discrete logarithm of a power of a literal generator power is its power
/// index.
theorem discrete_log_mod_with_order_generator_power_power_indexed(
    g: Nat, n: Nat, ord: Nat, k: Nat, m: Nat
) {
    is_multiplicative_order_mod(g, n, ord)
        implies discrete_log_mod_with_order(g, n, ord, g.pow(k).pow(m)) =
            discrete_log_mod_with_order_power_index(g, n, ord, g.pow(k), m)
} by {
    if is_multiplicative_order_mod(g, n, ord) {
        power_of_mod_pow(g, k, n)
        is_power_of_mod(g.pow(k), g, n)
        discrete_log_mod_with_order_power_indexed(g, n, ord, g.pow(k), m)
        discrete_log_mod_with_order(g, n, ord, g.pow(k).pow(m)) =
            discrete_log_mod_with_order_power_index(g, n, ord, g.pow(k), m)
    }
}

/// The discrete logarithm of a power is the logarithm multiplied by the
/// exponent, reduced modulo the order.
theorem discrete_log_mod_with_order_power(
    g: Nat, n: Nat, ord: Nat, a: Nat, m: Nat
) {
    is_multiplicative_order_mod(g, n, ord) and is_power_of_mod(a, g, n)
        implies discrete_log_mod_with_order(g, n, ord, a.pow(m)) =
            (discrete_log_mod_with_order(g, n, ord, a) * m).mod(ord)
} by {
    if is_multiplicative_order_mod(g, n, ord) and is_power_of_mod(a, g, n) {
        discrete_log_mod_with_order_power_indexed(g, n, ord, a, m)
        discrete_log_mod_with_order(g, n, ord, a.pow(m)) =
            discrete_log_mod_with_order_power_index(g, n, ord, a, m)
        discrete_log_mod_with_order_power_index_eq(g, n, ord, a, m)
        discrete_log_mod_with_order_power_index(g, n, ord, a, m) =
            (discrete_log_mod_with_order(g, n, ord, a) * m).mod(ord)
        discrete_log_mod_with_order(g, n, ord, a.pow(m)) =
            (discrete_log_mod_with_order(g, n, ord, a) * m).mod(ord)
    }
}

/// Under unit coverage, the discrete logarithm of a power of a unit is the
/// logarithm multiplied by the exponent, reduced modulo the order.
theorem unit_discrete_log_mod_with_order_power(
    g: Nat, n: Nat, ord: Nat, a: Nat, m: Nat
) {
    is_multiplicative_order_mod(g, n, ord) and powers_cover_units_mod(g, n)
        and a.coprime(n)
        implies discrete_log_mod_with_order(g, n, ord, a.pow(m)) =
            (discrete_log_mod_with_order(g, n, ord, a) * m).mod(ord)
} by {
    if is_multiplicative_order_mod(g, n, ord) and powers_cover_units_mod(g, n)
        and a.coprime(n) {
        powers_cover_units_mod_apply(g, n, a)
        is_power_of_mod(a, g, n)
        is_multiplicative_order_mod(g, n, ord) and is_power_of_mod(a, g, n)
        discrete_log_mod_with_order_power(g, n, ord, a, m)
    }
}

/// Multiplying a represented residue by a literal generator power adds the
/// reduced exponent of that generator power to the discrete logarithm.
theorem discrete_log_mod_with_order_mul_generator_power(
    g: Nat, n: Nat, ord: Nat, a: Nat, k: Nat
) {
    is_multiplicative_order_mod(g, n, ord) and is_power_of_mod(a, g, n)
        implies discrete_log_mod_with_order(g, n, ord, a * g.pow(k)) =
            (discrete_log_mod_with_order(g, n, ord, a) + k.mod(ord)).mod(ord)
} by {
    if is_multiplicative_order_mod(g, n, ord) and is_power_of_mod(a, g, n) {
        discrete_log_mod_with_order_mul_generator_power_indexed(g, n, ord, a, k)
        discrete_log_mod_with_order(g, n, ord, a * g.pow(k)) =
            discrete_log_mod_with_order_mul_index(g, n, ord, a, g.pow(k))
        discrete_log_mod_with_order_mul_index_eq(g, n, ord, a, g.pow(k))
        discrete_log_mod_with_order_mul_index(g, n, ord, a, g.pow(k)) =
            (discrete_log_mod_with_order(g, n, ord, a) +
                discrete_log_mod_with_order(g, n, ord, g.pow(k))).mod(ord)
        discrete_log_mod_with_order_of_power(g, n, ord, k)
        discrete_log_mod_with_order(g, n, ord, g.pow(k)) = k.mod(ord)
        (discrete_log_mod_with_order(g, n, ord, a) +
            discrete_log_mod_with_order(g, n, ord, g.pow(k))).mod(ord) =
            (discrete_log_mod_with_order(g, n, ord, a) + k.mod(ord)).mod(ord)
        discrete_log_mod_with_order(g, n, ord, a * g.pow(k)) =
            (discrete_log_mod_with_order(g, n, ord, a) + k.mod(ord)).mod(ord)
    }
}

/// Multiplying a literal generator power by a represented residue adds the
/// discrete logarithm after the reduced exponent of that generator power.
theorem discrete_log_mod_with_order_generator_power_mul(
    g: Nat, n: Nat, ord: Nat, k: Nat, a: Nat
) {
    is_multiplicative_order_mod(g, n, ord) and is_power_of_mod(a, g, n)
        implies discrete_log_mod_with_order(g, n, ord, g.pow(k) * a) =
            (k.mod(ord) + discrete_log_mod_with_order(g, n, ord, a)).mod(ord)
} by {
    if is_multiplicative_order_mod(g, n, ord) and is_power_of_mod(a, g, n) {
        discrete_log_mod_with_order_generator_power_mul_indexed(g, n, ord, k, a)
        discrete_log_mod_with_order(g, n, ord, g.pow(k) * a) =
            discrete_log_mod_with_order_mul_index(g, n, ord, g.pow(k), a)
        discrete_log_mod_with_order_mul_index_eq(g, n, ord, g.pow(k), a)
        discrete_log_mod_with_order_mul_index(g, n, ord, g.pow(k), a) =
            (discrete_log_mod_with_order(g, n, ord, g.pow(k)) +
                discrete_log_mod_with_order(g, n, ord, a)).mod(ord)
        discrete_log_mod_with_order_of_power(g, n, ord, k)
        discrete_log_mod_with_order(g, n, ord, g.pow(k)) = k.mod(ord)
        (discrete_log_mod_with_order(g, n, ord, g.pow(k)) +
            discrete_log_mod_with_order(g, n, ord, a)).mod(ord) =
            (k.mod(ord) + discrete_log_mod_with_order(g, n, ord, a)).mod(ord)
        discrete_log_mod_with_order(g, n, ord, g.pow(k) * a) =
            (k.mod(ord) + discrete_log_mod_with_order(g, n, ord, a)).mod(ord)
    }
}

/// The discrete logarithm of a product of two literal generator powers is the
/// sum of their reduced exponents.
theorem discrete_log_mod_with_order_generator_power_mul_generator_power(
    g: Nat, n: Nat, ord: Nat, i: Nat, j: Nat
) {
    is_multiplicative_order_mod(g, n, ord)
        implies discrete_log_mod_with_order(g, n, ord, g.pow(i) * g.pow(j)) =
            (i.mod(ord) + j.mod(ord)).mod(ord)
} by {
    if is_multiplicative_order_mod(g, n, ord) {
        discrete_log_mod_with_order_generator_power_mul_generator_power_indexed(
            g, n, ord, i, j)
        discrete_log_mod_with_order(g, n, ord, g.pow(i) * g.pow(j)) =
            discrete_log_mod_with_order_mul_index(g, n, ord, g.pow(i), g.pow(j))
        discrete_log_mod_with_order_mul_index_eq(g, n, ord, g.pow(i), g.pow(j))
        discrete_log_mod_with_order_mul_index(g, n, ord, g.pow(i), g.pow(j)) =
            (discrete_log_mod_with_order(g, n, ord, g.pow(i)) +
                discrete_log_mod_with_order(g, n, ord, g.pow(j))).mod(ord)
        discrete_log_mod_with_order_of_power(g, n, ord, i)
        discrete_log_mod_with_order(g, n, ord, g.pow(i)) = i.mod(ord)
        discrete_log_mod_with_order_of_power(g, n, ord, j)
        discrete_log_mod_with_order(g, n, ord, g.pow(j)) = j.mod(ord)
        (discrete_log_mod_with_order(g, n, ord, g.pow(i)) +
            discrete_log_mod_with_order(g, n, ord, g.pow(j))).mod(ord) =
            (i.mod(ord) + j.mod(ord)).mod(ord)
        discrete_log_mod_with_order(g, n, ord, g.pow(i) * g.pow(j)) =
            (i.mod(ord) + j.mod(ord)).mod(ord)
    }
}

/// The discrete logarithm of a power of a literal generator power is the
/// reduced exponent multiplied by the power exponent.
theorem discrete_log_mod_with_order_generator_power_power(
    g: Nat, n: Nat, ord: Nat, k: Nat, m: Nat
) {
    is_multiplicative_order_mod(g, n, ord)
        implies discrete_log_mod_with_order(g, n, ord, g.pow(k).pow(m)) =
            (k.mod(ord) * m).mod(ord)
} by {
    if is_multiplicative_order_mod(g, n, ord) {
        discrete_log_mod_with_order_generator_power_power_indexed(g, n, ord, k, m)
        discrete_log_mod_with_order(g, n, ord, g.pow(k).pow(m)) =
            discrete_log_mod_with_order_power_index(g, n, ord, g.pow(k), m)
        discrete_log_mod_with_order_power_index_eq(g, n, ord, g.pow(k), m)
        discrete_log_mod_with_order_power_index(g, n, ord, g.pow(k), m) =
            (discrete_log_mod_with_order(g, n, ord, g.pow(k)) * m).mod(ord)
        discrete_log_mod_with_order_of_power(g, n, ord, k)
        discrete_log_mod_with_order(g, n, ord, g.pow(k)) = k.mod(ord)
        (discrete_log_mod_with_order(g, n, ord, g.pow(k)) * m).mod(ord) =
            (k.mod(ord) * m).mod(ord)
        discrete_log_mod_with_order(g, n, ord, g.pow(k).pow(m)) =
            (k.mod(ord) * m).mod(ord)
    }
}

/// Under unit coverage, multiplying a unit by a literal generator power adds
/// the reduced exponent of that generator power to the discrete logarithm.
theorem unit_discrete_log_mod_with_order_mul_generator_power(
    g: Nat, n: Nat, ord: Nat, a: Nat, k: Nat
) {
    is_multiplicative_order_mod(g, n, ord) and powers_cover_units_mod(g, n)
        and a.coprime(n)
        implies discrete_log_mod_with_order(g, n, ord, a * g.pow(k)) =
            (discrete_log_mod_with_order(g, n, ord, a) + k.mod(ord)).mod(ord)
} by {
    if is_multiplicative_order_mod(g, n, ord) and powers_cover_units_mod(g, n)
        and a.coprime(n) {
        powers_cover_units_mod_apply(g, n, a)
        is_power_of_mod(a, g, n)
        discrete_log_mod_with_order_mul_generator_power(g, n, ord, a, k)
    }
}

/// Under unit coverage, multiplying a literal generator power by a unit adds
/// the discrete logarithm after the reduced exponent of that generator power.
theorem unit_discrete_log_mod_with_order_generator_power_mul(
    g: Nat, n: Nat, ord: Nat, k: Nat, a: Nat
) {
    is_multiplicative_order_mod(g, n, ord) and powers_cover_units_mod(g, n)
        and a.coprime(n)
        implies discrete_log_mod_with_order(g, n, ord, g.pow(k) * a) =
            (k.mod(ord) + discrete_log_mod_with_order(g, n, ord, a)).mod(ord)
} by {
    if is_multiplicative_order_mod(g, n, ord) and powers_cover_units_mod(g, n)
        and a.coprime(n) {
        powers_cover_units_mod_apply(g, n, a)
        is_power_of_mod(a, g, n)
        discrete_log_mod_with_order_generator_power_mul(g, n, ord, k, a)
    }
}
