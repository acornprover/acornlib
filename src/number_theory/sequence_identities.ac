// Classical integer sequence identities.
//
// This file collects the classical closed forms for finite sums over the
// naturals:
//
//   (a) the triangular numbers:            sum_{k=1}^{n} k     = n(n+1)/2
//   (b) the sum of squares:                sum_{k=1}^{n} k^2   = n(n+1)(2n+1)/6
//   (c) the sum of cubes:                  sum_{k=1}^{n} k^3   = (n(n+1)/2)^2
//   (d) the Fibonacci identities:          F_{n+2} = F_{n+1} + F_n and
//                                          sum_{k=1}^{n} F_k   = F_{n+2} - 1
//   (e) the geometric series of two:       sum_{k=0}^{n} 2^k   = 2^{n+1} - 1
//
// The power-sum closed forms (a)-(c) are proved in
// `src/number_theory/sum_of_powers.ac` (via Faulhaber-style recurrences) and
// are restated here with their sources cited.  The Fibonacci identities (d)
// and the geometric-series identity (e) are proved here; the geometric series
// is the p = 2 case of the finite geometric series in
// `src/number_theory/mersenne_perfect.ac`.

from nat import Nat, add_imp_sub, add_comm, add_assoc, suc_sub_one, mul_one_left
from rat import Rat
from pair import Pair
from list import partial
from data.nat.nat_range_sum import range_sum, range_sum_suc, range_sum_one
from data.nat.nat_range_sum_bridge import range_sum_eq_partial
from number_theory.sum_of_powers import power_sum, closed_first, closed_squares,
    closed_cubes, sum_of_first, sum_of_squares, sum_of_cubes, sum_of_cubes_square
from number_theory.mersenne_perfect import pow_sub_one_factor

numerals Nat
numerals Rat

// ---------------------------------------------------------------------------
// The triangular numbers: sum_{k=1}^{n} k = n(n+1)/2.
// ---------------------------------------------------------------------------

/// The sum of the first n positive integers is n(n+1)/2:
/// `sum_{k=1}^{n} k = n(n+1)/2`.
///
/// Restates `sum_of_first` from `number_theory/sum_of_powers.ac`, where the
/// sum is `power_sum(Nat.1, n) = 0^1 + 1^1 + ... + n^1` (the k = 0 term is
/// zero) and the closed form is `closed_first(n) = n(n+1)/2` in the rationals.
theorem seq_triangular_sum(n: Nat) {
    power_sum(Nat.1, n) = closed_first(n)
} by {
    sum_of_first(n)
    power_sum(Nat.1, n) = closed_first(n)
}

// ---------------------------------------------------------------------------
// The sum of the squares: sum_{k=1}^{n} k^2 = n(n+1)(2n+1)/6.
// ---------------------------------------------------------------------------

/// The sum of the first n squares is n(n+1)(2n+1)/6:
/// `sum_{k=1}^{n} k^2 = n(n+1)(2n+1)/6`.
///
/// Restates `sum_of_squares` from `number_theory/sum_of_powers.ac`, with the
/// closed form `closed_squares(n) = n(n+1)(2n+1)/6`.
theorem seq_sum_of_squares(n: Nat) {
    power_sum(Nat.2, n) = closed_squares(n)
} by {
    sum_of_squares(n)
    power_sum(Nat.2, n) = closed_squares(n)
}

// ---------------------------------------------------------------------------
// The sum of the cubes: sum_{k=1}^{n} k^3 = (n(n+1)/2)^2.
// ---------------------------------------------------------------------------

/// The sum of the first n cubes is (n(n+1)/2)^2:
/// `sum_{k=1}^{n} k^3 = (n(n+1)/2)^2`.
///
/// Restates `sum_of_cubes` from `number_theory/sum_of_powers.ac`, with the
/// closed form `closed_cubes(n) = (n(n+1)/2)^2`.
theorem seq_sum_of_cubes(n: Nat) {
    power_sum(Nat.3, n) = closed_cubes(n)
} by {
    sum_of_cubes(n)
    power_sum(Nat.3, n) = closed_cubes(n)
}

/// The sum of the first n cubes is the square of the sum of the first n
/// naturals: `sum_{k=1}^{n} k^3 = (sum_{k=1}^{n} k)^2` — the classical
/// "beautiful identity".
///
/// Restates `sum_of_cubes_square` from `number_theory/sum_of_powers.ac`.
theorem seq_sum_of_cubes_square(n: Nat) {
    power_sum(Nat.3, n) = power_sum(Nat.1, n).pow(Nat.2)
} by {
    sum_of_cubes_square(n)
    power_sum(Nat.3, n) = power_sum(Nat.1, n).pow(Nat.2)
}

// ---------------------------------------------------------------------------
// The Fibonacci numbers.
// ---------------------------------------------------------------------------

/// A pair `(F_n, F_{n+1})` used to define the Fibonacci numbers iteratively,
/// matching `fib_pair` of `combinatorics/generating_functions.ac`.
define seq_fib_pair(n: Nat) -> Pair[Nat, Nat] {
    match n {
        Nat.zero {
            Pair.new(Nat.0, Nat.1)
        }
        Nat.suc(k) {
            let p: Pair[Nat, Nat] = seq_fib_pair(k)
            Pair.new(p.second, p.first + p.second)
        }
    }
}

/// The n-th Fibonacci number, with `F_0 = 0` and `F_1 = 1`.
///
/// The library's `fib` lives in `combinatorics/generating_functions.ac`, whose
/// module is private to the combinatorics package (only that package's
/// `interface.ac` is importable from outside), so the sequence is restated
/// here with the same pair iteration and base values; the defining recurrence
/// is proved below as `seq_fib_suc_suc`.
define seq_fib(n: Nat) -> Nat {
    seq_fib_pair(n).first
}

/// The successor step of the Fibonacci pair iteration.
theorem seq_fib_pair_suc(k: Nat) {
    seq_fib_pair(k.suc) =
        Pair.new(seq_fib_pair(k).second, seq_fib_pair(k).first + seq_fib_pair(k).second)
}

/// The base value of the Fibonacci pair iteration.
theorem seq_fib_pair_zero {
    seq_fib_pair(Nat.0) = Pair.new(Nat.0, Nat.1)
}

/// The second component of the Fibonacci pair is the next Fibonacci number.
theorem seq_fib_pair_second_suc(k: Nat) {
    seq_fib_pair(k).second = seq_fib(k.suc)
} by {
    seq_fib(k.suc) = seq_fib_pair(k.suc).first
    seq_fib_pair_suc(k)
    seq_fib_pair(k.suc).first =
        Pair.new(seq_fib_pair(k).second, seq_fib_pair(k).first + seq_fib_pair(k).second).first
    Pair.new(seq_fib_pair(k).second, seq_fib_pair(k).first + seq_fib_pair(k).second).first =
        seq_fib_pair(k).second
    seq_fib_pair(k).second = seq_fib(k.suc)
}

/// The zero-th Fibonacci number is zero: `F_0 = 0`.
theorem seq_fib_zero {
    seq_fib(Nat.0) = Nat.0
}

/// The first Fibonacci number is one: `F_1 = 1`.
theorem seq_fib_one {
    seq_fib(Nat.1) = Nat.1
} by {
    seq_fib(Nat.1) = seq_fib_pair(Nat.1).first
    seq_fib_pair_suc(Nat.0)
    seq_fib_pair(Nat.1).first =
        Pair.new(seq_fib_pair(Nat.0).second, seq_fib_pair(Nat.0).first + seq_fib_pair(Nat.0).second).first
    Pair.new(seq_fib_pair(Nat.0).second, seq_fib_pair(Nat.0).first + seq_fib_pair(Nat.0).second).first =
        seq_fib_pair(Nat.0).second
    seq_fib_pair(Nat.0).second = Pair.new(Nat.0, Nat.1).second
    Pair.new(Nat.0, Nat.1).second = Nat.1
    seq_fib(Nat.1) = Nat.1
}

/// The defining recurrence of the Fibonacci numbers: `F_{n+2} = F_{n+1} + F_n`.
///
/// This is `fib_suc_suc` of `combinatorics/generating_functions.ac`, restated
/// for the local copy of the sequence.
theorem seq_fib_suc_suc(j: Nat) {
    seq_fib(j.suc.suc) = seq_fib(j.suc) + seq_fib(j)
} by {
    seq_fib(j.suc.suc) = seq_fib_pair(j.suc.suc).first
    seq_fib_pair_suc(j.suc)
    seq_fib_pair(j.suc.suc).first =
        Pair.new(seq_fib_pair(j.suc).second, seq_fib_pair(j.suc).first + seq_fib_pair(j.suc).second).first
    Pair.new(seq_fib_pair(j.suc).second, seq_fib_pair(j.suc).first + seq_fib_pair(j.suc).second).first =
        seq_fib_pair(j.suc).second
    seq_fib_pair_second_suc(j)
    seq_fib_pair(j).second = seq_fib(j.suc)
    seq_fib_pair_suc(j)
    seq_fib_pair(j.suc).second =
        Pair.new(seq_fib_pair(j).second, seq_fib_pair(j).first + seq_fib_pair(j).second).second
    Pair.new(seq_fib_pair(j).second, seq_fib_pair(j).first + seq_fib_pair(j).second).second =
        seq_fib_pair(j).first + seq_fib_pair(j).second
    seq_fib_pair(j).first = seq_fib(j)
    seq_fib(j.suc.suc) = seq_fib(j) + seq_fib(j.suc)
    seq_fib(j) + seq_fib(j.suc) = seq_fib(j.suc) + seq_fib(j)
    seq_fib(j.suc.suc) = seq_fib(j.suc) + seq_fib(j)
}

/// The second Fibonacci number is one: `F_2 = 1`.
theorem seq_fib_two {
    seq_fib(Nat.2) = Nat.1
} by {
    seq_fib_suc_suc(Nat.0)
    seq_fib(Nat.0.suc.suc) = seq_fib(Nat.0.suc) + seq_fib(Nat.0)
    seq_fib_zero
    seq_fib_one
    seq_fib(Nat.2) = Nat.1 + Nat.0
    Nat.1 + Nat.0 = Nat.1
    seq_fib(Nat.2) = Nat.1
}

/// The sum of the first n + 1 Fibonacci numbers is one more than `F_{n+2}`:
/// `F_0 + F_1 + ... + F_n + 1 = F_{n+2}`.
///
/// This is the induction-friendly form of the summation identity; the closed
/// form `sum_{k=1}^{n} F_k = F_{n+2} - 1` follows from it by
/// `add_imp_sub` (the k = 0 term `F_0 = 0` contributes nothing).
theorem seq_fib_sum_suc(n: Nat) {
    range_sum(seq_fib, n.suc) + Nat.1 = seq_fib(n.suc.suc)
} by {
    define p(k: Nat) -> Bool {
        range_sum(seq_fib, k.suc) + Nat.1 = seq_fib(k.suc.suc)
    }
    range_sum_one(seq_fib)
    range_sum(seq_fib, Nat.1) = seq_fib(Nat.0)
    seq_fib_zero
    seq_fib(Nat.0) = Nat.0
    range_sum(seq_fib, Nat.1) = Nat.0
    seq_fib_two
    seq_fib(Nat.2) = Nat.1
    range_sum(seq_fib, Nat.1) + Nat.1 = seq_fib(Nat.2)
    p(Nat.0)
    forall(k: Nat) {
        if p(k) {
            range_sum(seq_fib, k.suc) + Nat.1 = seq_fib(k.suc.suc)
            range_sum_suc(seq_fib, k.suc)
            range_sum(seq_fib, k.suc.suc) = range_sum(seq_fib, k.suc) + seq_fib(k.suc)
            range_sum(seq_fib, k.suc.suc) + Nat.1 =
                (range_sum(seq_fib, k.suc) + seq_fib(k.suc)) + Nat.1
            add_assoc(range_sum(seq_fib, k.suc), seq_fib(k.suc), Nat.1)
            (range_sum(seq_fib, k.suc) + seq_fib(k.suc)) + Nat.1 =
                range_sum(seq_fib, k.suc) + (seq_fib(k.suc) + Nat.1)
            add_comm(seq_fib(k.suc), Nat.1)
            seq_fib(k.suc) + Nat.1 = Nat.1 + seq_fib(k.suc)
            range_sum(seq_fib, k.suc) + (seq_fib(k.suc) + Nat.1) =
                range_sum(seq_fib, k.suc) + (Nat.1 + seq_fib(k.suc))
            add_assoc(range_sum(seq_fib, k.suc), Nat.1, seq_fib(k.suc))
            (range_sum(seq_fib, k.suc) + Nat.1) + seq_fib(k.suc) =
                range_sum(seq_fib, k.suc) + (Nat.1 + seq_fib(k.suc))
            (range_sum(seq_fib, k.suc) + seq_fib(k.suc)) + Nat.1 =
                (range_sum(seq_fib, k.suc) + Nat.1) + seq_fib(k.suc)
            (range_sum(seq_fib, k.suc) + Nat.1) + seq_fib(k.suc) =
                seq_fib(k.suc.suc) + seq_fib(k.suc)
            seq_fib_suc_suc(k.suc)
            seq_fib(k.suc.suc.suc) = seq_fib(k.suc.suc) + seq_fib(k.suc)
            range_sum(seq_fib, k.suc.suc) + Nat.1 = seq_fib(k.suc.suc.suc)
            p(k.suc)
        }
    }
    p(Nat.0) and forall(k: Nat) {
        p(k) implies p(k.suc)
    }
    Nat.induction(p)
    p(n)
}

/// The classical summation identity for the Fibonacci numbers:
/// `sum_{k=1}^{n} F_k = F_{n+2} - 1`.
///
/// The sum on the left is `range_sum(seq_fib, n.suc) = F_0 + F_1 + ... + F_n`,
/// and `F_0 = 0`, so the k = 0 term contributes nothing; the subtraction is
/// the truncated natural subtraction, and `F_{n+2} >= 1` always holds.
theorem seq_fib_sum(n: Nat) {
    range_sum(seq_fib, n.suc) = seq_fib(n.suc.suc) - Nat.1
} by {
    seq_fib_sum_suc(n)
    range_sum(seq_fib, n.suc) + Nat.1 = seq_fib(n.suc.suc)
    add_imp_sub(range_sum(seq_fib, n.suc), Nat.1, seq_fib(n.suc.suc))
    seq_fib(n.suc.suc) - Nat.1 = range_sum(seq_fib, n.suc)
    range_sum(seq_fib, n.suc) = seq_fib(n.suc.suc) - Nat.1
}

// ---------------------------------------------------------------------------
// The geometric series: sum_{k=0}^{n} 2^k = 2^{n+1} - 1.
// ---------------------------------------------------------------------------

/// The finite geometric series of the powers of two, in partial-sum form:
/// `2^0 + 2^1 + ... + 2^n = 2^{n+1} - 1`.
///
/// This is the p = 2 case of `pow_sub_one_factor` from
/// `number_theory/mersenne_perfect.ac`, which states
/// `p^k - 1 = (p - 1) * (p^0 + ... + p^(k-1))` for `p > 1`.
theorem seq_geom_pow2_partial(n: Nat) {
    partial(Nat.2.pow, n.suc) = Nat.2.pow(n.suc) - Nat.1
} by {
    Nat.1 < Nat.2
    pow_sub_one_factor(Nat.2, n.suc)
    Nat.2.pow(n.suc) - Nat.1 = (Nat.2 - Nat.1) * partial(Nat.2.pow, n.suc)
    suc_sub_one(Nat.1)
    Nat.1.suc - Nat.1 = Nat.1
    Nat.1.suc = Nat.2
    Nat.2 - Nat.1 = Nat.1
    mul_one_left(partial(Nat.2.pow, n.suc))
    Nat.1 * partial(Nat.2.pow, n.suc) = partial(Nat.2.pow, n.suc)
    Nat.2.pow(n.suc) - Nat.1 = partial(Nat.2.pow, n.suc)
    partial(Nat.2.pow, n.suc) = Nat.2.pow(n.suc) - Nat.1
}

/// The geometric series in range-sum form:
/// `sum_{k=0}^{n} 2^k = 2^{n+1} - 1`.
///
/// `range_sum(Nat.2.pow, n.suc)` sums `2^0, 2^1, ..., 2^n`, and the bridge
/// `range_sum_eq_partial` identifies it with the partial sum above.
theorem seq_sum_pow2(n: Nat) {
    range_sum(Nat.2.pow, n.suc) = Nat.2.pow(n.suc) - Nat.1
} by {
    range_sum_eq_partial(Nat.2.pow, n.suc)
    range_sum(Nat.2.pow, n.suc) = partial(Nat.2.pow, n.suc)
    seq_geom_pow2_partial(n)
    partial(Nat.2.pow, n.suc) = Nat.2.pow(n.suc) - Nat.1
    range_sum(Nat.2.pow, n.suc) = Nat.2.pow(n.suc) - Nat.1
}
