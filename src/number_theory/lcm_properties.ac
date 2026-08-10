from nat import Nat, gcd_mul_lcm, mul_cancel_left, mul_to_zero, mul_comm,
    mul_one_left, mul_one_right, gcd_divides_left, gcd_divides_right, divides_gcd,
    divides_trans, divides_mul_right, divides_mul, div_mul, divides_symm, divides_self,
    gcd_nonzero_left, cofactor, gcd_mult_left, gcd_zero_left, gcd_zero_right
from number_theory.coprime import coprime_divides_of_divides_mul, coprime_comm,
    coprime_of_divisors
from number_theory.lcm import lcm_divides_left, lcm_divides_right, lcm_divides_of_common,
    lcm_zero_left, lcm_zero_right
numerals Nat

/// The product of the gcd and the lcm of two naturals is their product.
/// Restates `gcd_mul_lcm` from `nat` so that this file is self-contained.
theorem lcm_gcd_product(a: Nat, b: Nat) {
    a.gcd(b) * a.lcm(b) = a * b
} by {
    gcd_mul_lcm(a, b)
    a.gcd(b) * a.lcm(b) = a * b
}

/// The lcm of two naturals is a common multiple from the left.
theorem lcm_common_multiple_left(a: Nat, b: Nat) {
    a.divides(a.lcm(b))
} by {
    lcm_divides_left(a, b)
    a.divides(a.lcm(b))
}

/// The lcm of two naturals is a common multiple from the right.
theorem lcm_common_multiple_right(a: Nat, b: Nat) {
    b.divides(a.lcm(b))
} by {
    lcm_divides_right(a, b)
    b.divides(a.lcm(b))
}

/// The lcm of two naturals divides every common multiple: the least-common-
/// multiple property.
theorem lcm_divides_common_multiple(a: Nat, b: Nat, c: Nat) {
    a.divides(c) and b.divides(c) implies a.lcm(b).divides(c)
} by {
    lcm_divides_of_common(a, b, c)
    a.divides(c) and b.divides(c) implies a.lcm(b).divides(c)
}

/// If the gcd of two naturals is one, their lcm is their product.
theorem lcm_eq_mul_of_gcd_one(a: Nat, b: Nat) {
    a.gcd(b) = Nat.1 implies a.lcm(b) = a * b
} by {
    if a.gcd(b) = Nat.1 {
        gcd_mul_lcm(a, b)
        a.gcd(b) * a.lcm(b) = a * b
        Nat.1 * a.lcm(b) = a * b
        a.lcm(b) = a * b
    }
}

/// If the lcm of two nonzero naturals is their product, their gcd is one.
theorem gcd_one_of_lcm_eq_mul(a: Nat, b: Nat) {
    a != Nat.0 and b != Nat.0 and a.lcm(b) = a * b implies a.gcd(b) = Nat.1
} by {
    if a != Nat.0 and b != Nat.0 and a.lcm(b) = a * b {
        gcd_mul_lcm(a, b)
        a.gcd(b) * a.lcm(b) = a * b
        a.gcd(b) * (a * b) = a * b
        a.gcd(b) * (a * b) = (a * b) * a.gcd(b)
        (a * b) * Nat.1 = a * b
        (a * b) * a.gcd(b) = (a * b) * Nat.1
        if a * b = Nat.0 {
            mul_to_zero(a, b)
            a = Nat.0 or b = Nat.0
            false
        }
        a * b != Nat.0
        mul_cancel_left(a * b, a.gcd(b), Nat.1)
        a.gcd(b) = Nat.1
    }
}

/// For nonzero naturals, coprimality (gcd one) is equivalent to the lcm
/// being the product. The nonzero hypotheses are necessary: lcm(0, b) = 0
/// equals 0 * b for every b, while gcd(0, b) = b is one only for b = 1.
theorem lcm_eq_mul_iff_gcd_one(a: Nat, b: Nat) {
    a != Nat.0 and b != Nat.0 implies ((a.gcd(b) = Nat.1) = (a.lcm(b) = a * b))
} by {
    if a != Nat.0 and b != Nat.0 {
        if a.gcd(b) = Nat.1 {
            lcm_eq_mul_of_gcd_one(a, b)
            a.lcm(b) = a * b
        }
        if a.lcm(b) = a * b {
            gcd_one_of_lcm_eq_mul(a, b)
            a.gcd(b) = Nat.1
        }
        (a.gcd(b) = Nat.1) = (a.lcm(b) = a * b)
    }
}

// The distributive law for gcd over lcm.  The two divisibility directions
// are proved separately and combined with `divides_symm`.

/// The lcm of the gcds divides the gcd with the lcm.
theorem gcd_lcm_distrib_left(a: Nat, b: Nat, c: Nat) {
    (a.gcd(b)).lcm(a.gcd(c)).divides(a.gcd(b.lcm(c)))
} by {
    gcd_divides_right(a, b)
    gcd_divides_left(a, b)
    a.gcd(b).divides(a)
    a.gcd(b).divides(b)
    lcm_divides_left(b, c)
    b.divides(b.lcm(c))
    divides_trans(a.gcd(b), b, b.lcm(c))
    a.gcd(b).divides(b.lcm(c))
    divides_gcd(a.gcd(b), a, b.lcm(c))
    a.gcd(b).divides(a.gcd(b.lcm(c)))
    gcd_divides_right(a, c)
    gcd_divides_left(a, c)
    a.gcd(c).divides(a)
    a.gcd(c).divides(c)
    lcm_divides_right(b, c)
    c.divides(b.lcm(c))
    divides_trans(a.gcd(c), c, b.lcm(c))
    a.gcd(c).divides(b.lcm(c))
    divides_gcd(a.gcd(c), a, b.lcm(c))
    a.gcd(c).divides(a.gcd(b.lcm(c)))
    lcm_divides_of_common(a.gcd(b), a.gcd(c), a.gcd(b.lcm(c)))
    (a.gcd(b)).lcm(a.gcd(c)).divides(a.gcd(b.lcm(c)))
}

/// If x1 divides y1 and x2 divides y2, then their product divides the product.
theorem prod_divides(a1: Nat, a2: Nat, b1: Nat, b2: Nat) {
    a1.divides(b1) and a2.divides(b2) implies (a1 * a2).divides(b1 * b2)
} by {
    if a1.divides(b1) and a2.divides(b2) {
        divides_mul_right(a1, b1, a2)
        (a1 * a2).divides(b1 * a2)
        divides_mul_right(a2, b2, b1)
        (a2 * b1).divides(b2 * b1)
        b1 * a2 = a2 * b1
        b2 * b1 = b1 * b2
        (a1 * a2).divides(b1 * a2)
        (a1 * a2).divides(a2 * b1)
        divides_trans(a1 * a2, a2 * b1, b1 * b2)
        (a1 * a2).divides(b1 * b2)
    }
}

/// Coprime divisors of a common multiple multiply to a divisor.
theorem coprime_mul_divides(p: Nat, q: Nat, m: Nat) {
    p.coprime(q) and p.divides(m) and q.divides(m) implies (p * q).divides(m)
} by {
    if p.coprime(q) and p.divides(m) and q.divides(m) {
        let k: Nat satisfy { p * k = m }
        q.divides(p * k)
        coprime_comm(p, q)
        q.coprime(p)
        coprime_divides_of_divides_mul(q, p, k)
        q.divides(k)
        let j: Nat satisfy { q * j = k }
        p * q * j = q * (p * j)
        q * (p * j) = p * (q * j)
        p * (q * j) = p * k
        p * k = m
        p * q * j = p * (q * j)
        p * q * j = m
        (p * q).divides(m)
    }
}

/// Gauss's lemma for divisibility: if d divides x * y then d / gcd(d, x)
/// divides y.
theorem div_of_divides_mul(d: Nat, x: Nat, y: Nat) {
    d != Nat.0 and d.divides(x * y) implies (d.div(d.gcd(x))).divides(y)
} by {
    if d != Nat.0 and d.divides(x * y) {
        gcd_nonzero_left(d, x)
        d.gcd(x) != Nat.0
        gcd_divides_left(d, x)
        gcd_divides_right(d, x)
        let dp: Nat satisfy { d.gcd(x) * dp = d }
        let xp: Nat satisfy { d.gcd(x) * xp = x }
        dp * d.gcd(x) = d.gcd(x) * dp
        xp * d.gcd(x) = d.gcd(x) * xp
        dp * d.gcd(x) = d
        xp * d.gcd(x) = x
        cofactor(d, x, dp, xp)
        dp.gcd(xp) = Nat.1
        let w: Nat satisfy { d * w = x * y }
        d.gcd(x) * dp = d
        d.gcd(x) * dp * w = d * w
        d * w = x * y
        d.gcd(x) * dp * w = x * y
        d.gcd(x) * xp = x
        d.gcd(x) * xp * y = x * y
        d.gcd(x) * dp * w = d.gcd(x) * xp * y
        d.gcd(x) * (dp * w) = d.gcd(x) * (xp * y)
        mul_cancel_left(d.gcd(x), dp * w, xp * y)
        dp * w = xp * y
        dp.coprime(xp)
        coprime_divides_of_divides_mul(dp, xp, y)
        dp.divides(y)
        div_mul(dp, d.gcd(x))
        (dp * d.gcd(x)).div(d.gcd(x)) = dp
        dp * d.gcd(x) = d
        d.div(d.gcd(x)) = dp
        (d.div(d.gcd(x))).divides(y)
    }
}

/// A factor coprime to a can be dropped from a product inside gcd(a, -).
theorem gcd_coprime_factor(a: Nat, g: Nat, t: Nat) {
    a.coprime(g) implies a.gcd(g * t) = a.gcd(t)
} by {
    if a.coprime(g) {
        gcd_divides_left(a, t)
        a.gcd(t).divides(a)
        a.gcd(t).divides(t)
        divides_mul(t, g, a.gcd(t))
        a.gcd(t).divides(t * g)
        t * g = g * t
        a.gcd(t).divides(g * t)
        divides_gcd(a.gcd(t), a, g * t)
        a.gcd(t).divides(a.gcd(g * t))
        gcd_divides_left(a, g * t)
        a.gcd(g * t).divides(a)
        gcd_divides_right(a, g * t)
        a.gcd(g * t).divides(g * t)
        coprime_of_divisors(a, g, a.gcd(g * t), g)
        a.gcd(g * t).coprime(g)
        coprime_divides_of_divides_mul(a.gcd(g * t), g, t)
        a.gcd(g * t).divides(t)
        divides_gcd(a.gcd(g * t), a, t)
        a.gcd(g * t).divides(a.gcd(t))
        divides_symm(a.gcd(t), a.gcd(g * t))
        a.gcd(t) = a.gcd(g * t)
        a.gcd(g * t) = a.gcd(t)
    }
}

/// The gcd with a product of coprime factors is the product of the gcds.
theorem gcd_mul_coprime(a: Nat, x: Nat, y: Nat) {
    x.coprime(y) implies a.gcd(x * y) = a.gcd(x) * a.gcd(y)
} by {
    if x.coprime(y) {
        gcd_divides_left(a, x)
        a.gcd(x).divides(a)
        gcd_divides_right(a, x)
        a.gcd(x).divides(x)
        divides_mul(x, y, a.gcd(x))
        a.gcd(x).divides(x * y)
        divides_gcd(a.gcd(x), a, x * y)
        a.gcd(x).divides(a.gcd(x * y))
        gcd_divides_left(a, y)
        a.gcd(y).divides(a)
        gcd_divides_right(a, y)
        a.gcd(y).divides(y)
        divides_mul(y, x, a.gcd(y))
        a.gcd(y).divides(y * x)
        y * x = x * y
        a.gcd(y).divides(x * y)
        divides_gcd(a.gcd(y), a, x * y)
        a.gcd(y).divides(a.gcd(x * y))
        coprime_of_divisors(x, y, a.gcd(x), a.gcd(y))
        a.gcd(x).coprime(a.gcd(y))
        coprime_mul_divides(a.gcd(x), a.gcd(y), a.gcd(x * y))
        (a.gcd(x) * a.gcd(y)).divides(a.gcd(x * y))
        if a.gcd(x * y) = Nat.0 {
            gcd_nonzero_left(a, x * y)
            a = Nat.0
            gcd_zero_left(x * y)
            a.gcd(x * y) = x * y
            gcd_zero_left(x)
            gcd_zero_left(y)
            a.gcd(x) = x
            a.gcd(y) = y
            a.gcd(x) * a.gcd(y) = x * y
            a.gcd(x * y).divides(a.gcd(x) * a.gcd(y))
        } else {
            gcd_divides_left(a, x * y)
            a.gcd(x * y).divides(a)
            gcd_divides_right(a, x * y)
            a.gcd(x * y).divides(x * y)
            div_of_divides_mul(a.gcd(x * y), x, y)
            (a.gcd(x * y)).div((a.gcd(x * y)).gcd(x)).divides(y)
            gcd_nonzero_left(a, x * y)
            a.gcd(x * y) != Nat.0
            gcd_divides_left(a.gcd(x * y), x)
            gcd_divides_right(a.gcd(x * y), x)
            let q: Nat satisfy { (a.gcd(x * y)).gcd(x) * q = a.gcd(x * y) }
            (a.gcd(x * y)).gcd(x) * q = a.gcd(x * y)
            div_mul(q, (a.gcd(x * y)).gcd(x))
            (q * (a.gcd(x * y)).gcd(x)).div((a.gcd(x * y)).gcd(x)) = q
            q * (a.gcd(x * y)).gcd(x) = (a.gcd(x * y)).gcd(x) * q
            ((a.gcd(x * y)).gcd(x) * q).div((a.gcd(x * y)).gcd(x)) = q
            (a.gcd(x * y)).div((a.gcd(x * y)).gcd(x)) = q
            q.divides(y)
            q.divides(a.gcd(x * y))
            q.divides(a)
            divides_gcd(q, a, y)
            q.divides(a.gcd(y))
            (a.gcd(x * y)).gcd(x).divides(a.gcd(x * y))
            (a.gcd(x * y)).gcd(x).divides(a)
            divides_gcd((a.gcd(x * y)).gcd(x), a, x)
            (a.gcd(x * y)).gcd(x).divides(a.gcd(x))
            prod_divides((a.gcd(x * y)).gcd(x), q, a.gcd(x), a.gcd(y))
            ((a.gcd(x * y)).gcd(x) * q).divides(a.gcd(x) * a.gcd(y))
            (a.gcd(x * y)).gcd(x) * q = a.gcd(x * y)
            a.gcd(x * y).divides(a.gcd(x) * a.gcd(y))
        }
        divides_symm(a.gcd(x) * a.gcd(y), a.gcd(x * y))
        a.gcd(x * y) = a.gcd(x) * a.gcd(y)
    }
}

/// The gcd of two gcds of the same number is the gcd with their gcd.
theorem gcd_assoc(a: Nat, x: Nat, y: Nat) {
    (a.gcd(x)).gcd(a.gcd(y)) = a.gcd(x.gcd(y))
} by {
    gcd_divides_left((a.gcd(x)), a.gcd(y))
    (a.gcd(x)).gcd(a.gcd(y)).divides(a.gcd(x))
    gcd_divides_left(a, x)
    a.gcd(x).divides(a)
    divides_trans((a.gcd(x)).gcd(a.gcd(y)), a.gcd(x), a)
    (a.gcd(x)).gcd(a.gcd(y)).divides(a)
    gcd_divides_right(a, x)
    a.gcd(x).divides(x)
    divides_trans((a.gcd(x)).gcd(a.gcd(y)), a.gcd(x), x)
    (a.gcd(x)).gcd(a.gcd(y)).divides(x)
    gcd_divides_right((a.gcd(x)), a.gcd(y))
    (a.gcd(x)).gcd(a.gcd(y)).divides(a.gcd(y))
    gcd_divides_right(a, y)
    a.gcd(y).divides(y)
    divides_trans((a.gcd(x)).gcd(a.gcd(y)), a.gcd(y), y)
    (a.gcd(x)).gcd(a.gcd(y)).divides(y)
    divides_gcd((a.gcd(x)).gcd(a.gcd(y)), x, y)
    (a.gcd(x)).gcd(a.gcd(y)).divides(x.gcd(y))
    divides_gcd((a.gcd(x)).gcd(a.gcd(y)), a, x.gcd(y))
    (a.gcd(x)).gcd(a.gcd(y)).divides(a.gcd(x.gcd(y)))
    gcd_divides_left(a, x.gcd(y))
    a.gcd(x.gcd(y)).divides(a)
    a.gcd(x.gcd(y)).divides(x.gcd(y))
    gcd_divides_left(x, y)
    x.gcd(y).divides(x)
    divides_trans(a.gcd(x.gcd(y)), x.gcd(y), x)
    a.gcd(x.gcd(y)).divides(x)
    divides_gcd(a.gcd(x.gcd(y)), a, x)
    a.gcd(x.gcd(y)).divides(a.gcd(x))
    gcd_divides_right(x, y)
    x.gcd(y).divides(y)
    divides_trans(a.gcd(x.gcd(y)), x.gcd(y), y)
    a.gcd(x.gcd(y)).divides(y)
    divides_gcd(a.gcd(x.gcd(y)), a, y)
    a.gcd(x.gcd(y)).divides(a.gcd(y))
    divides_gcd(a.gcd(x.gcd(y)), a.gcd(x), a.gcd(y))
    a.gcd(x.gcd(y)).divides((a.gcd(x)).gcd(a.gcd(y)))
    divides_symm((a.gcd(x)).gcd(a.gcd(y)), a.gcd(x.gcd(y)))
    (a.gcd(x)).gcd(a.gcd(y)) = a.gcd(x.gcd(y))
}

/// The lcm of x and y equals g * xp * yp where g = gcd(x, y), x = g * xp and
/// y = g * yp.
theorem lcm_cofactor(x: Nat, y: Nat, xp: Nat, yp: Nat) {
    x.gcd(y) != Nat.0 and x.gcd(y) * xp = x and x.gcd(y) * yp = y implies
        x.gcd(y) * xp * yp = x.lcm(y)
} by {
    if x.gcd(y) != Nat.0 and x.gcd(y) * xp = x and x.gcd(y) * yp = y {
        gcd_mul_lcm(x, y)
        x.gcd(y) * x.lcm(y) = x * y
        x * y = (x.gcd(y) * xp) * (x.gcd(y) * yp)
        (x.gcd(y) * xp) * (x.gcd(y) * yp) = x.gcd(y) * (xp * (x.gcd(y) * yp))
        xp * (x.gcd(y) * yp) = (xp * x.gcd(y)) * yp
        xp * x.gcd(y) = x.gcd(y) * xp
        (xp * x.gcd(y)) * yp = (x.gcd(y) * xp) * yp
        xp * (x.gcd(y) * yp) = (x.gcd(y) * xp) * yp
        x.gcd(y) * (xp * (x.gcd(y) * yp)) = x.gcd(y) * ((x.gcd(y) * xp) * yp)
        x.gcd(y) * ((x.gcd(y) * xp) * yp) = x.gcd(y) * (x.gcd(y) * xp * yp)
        x.gcd(y) * x.lcm(y) = x.gcd(y) * (x.gcd(y) * xp * yp)
        mul_cancel_left(x.gcd(y), x.lcm(y), x.gcd(y) * xp * yp)
        x.lcm(y) = x.gcd(y) * xp * yp
        x.gcd(y) * xp * yp = x.lcm(y)
    }
}

/// The map gcd(a, -) respects the gcd/lcm lattice structure:
/// gcd(a, x) * gcd(a, y) = gcd(a, gcd(x, y)) * gcd(a, lcm(x, y)).
theorem gcd_lcm_identity(a: Nat, x: Nat, y: Nat) {
    a.gcd(x) * a.gcd(y) = a.gcd(x.gcd(y)) * a.gcd(x.lcm(y))
} by {
    if x = Nat.0 {
        gcd_zero_right(a)
        a.gcd(Nat.0) = a
        gcd_zero_left(y)
        x.gcd(y) = y
        lcm_zero_left(y)
        x.lcm(y) = Nat.0
        gcd_zero_right(a)
        a.gcd(x.lcm(y)) = a
        a.gcd(x) * a.gcd(y) = a * a.gcd(y)
        a.gcd(x.gcd(y)) * a.gcd(x.lcm(y)) = a.gcd(y) * a
        a * a.gcd(y) = a.gcd(y) * a
        a.gcd(x) * a.gcd(y) = a.gcd(x.gcd(y)) * a.gcd(x.lcm(y))
    } else {
        if y = Nat.0 {
            gcd_zero_right(a)
            a.gcd(Nat.0) = a
            gcd_zero_left(x)
            x.gcd(y) = x
            lcm_zero_right(x)
            x.lcm(y) = Nat.0
            gcd_zero_right(a)
            a.gcd(x.lcm(y)) = a
            a.gcd(x) * a.gcd(y) = a.gcd(x) * a
            a.gcd(x.gcd(y)) * a.gcd(x.lcm(y)) = a.gcd(x) * a
            a.gcd(x) * a.gcd(y) = a.gcd(x.gcd(y)) * a.gcd(x.lcm(y))
        } else {
            gcd_nonzero_left(x, y)
            x.gcd(y) != Nat.0
            gcd_divides_left(x, y)
            gcd_divides_right(x, y)
            let xp: Nat satisfy { x.gcd(y) * xp = x }
            let yp: Nat satisfy { x.gcd(y) * yp = y }
            xp * x.gcd(y) = x.gcd(y) * xp
            yp * x.gcd(y) = x.gcd(y) * yp
            xp * x.gcd(y) = x
            yp * x.gcd(y) = y
            cofactor(x, y, xp, yp)
            xp.gcd(yp) = Nat.1
            xp.coprime(yp)
            lcm_cofactor(x, y, xp, yp)
            x.gcd(y) * xp * yp = x.lcm(y)
            if a = Nat.0 {
                gcd_zero_left(x)
                gcd_zero_left(y)
                gcd_zero_left(x.gcd(y))
                gcd_zero_left(x.lcm(y))
                a.gcd(x) = x
                a.gcd(y) = y
                a.gcd(x.gcd(y)) = x.gcd(y)
                a.gcd(x.lcm(y)) = x.lcm(y)
                a.gcd(x) * a.gcd(y) = x * y
                a.gcd(x.gcd(y)) * a.gcd(x.lcm(y)) = x.gcd(y) * x.lcm(y)
                gcd_mul_lcm(x, y)
                x.gcd(y) * x.lcm(y) = x * y
                x.gcd(y) * x.lcm(y) = a.gcd(x.gcd(y)) * a.gcd(x.lcm(y))
                a.gcd(x) * a.gcd(y) = a.gcd(x.gcd(y)) * a.gcd(x.lcm(y))
            } else {
                gcd_nonzero_left(a, x.gcd(y))
                a.gcd(x.gcd(y)) != Nat.0
                gcd_divides_left(a, x.gcd(y))
                gcd_divides_right(a, x.gcd(y))
                let ap: Nat satisfy { a.gcd(x.gcd(y)) * ap = a }
                let gp: Nat satisfy { a.gcd(x.gcd(y)) * gp = x.gcd(y) }
                ap * a.gcd(x.gcd(y)) = a.gcd(x.gcd(y)) * ap
                gp * a.gcd(x.gcd(y)) = a.gcd(x.gcd(y)) * gp
                ap * a.gcd(x.gcd(y)) = a
                gp * a.gcd(x.gcd(y)) = x.gcd(y)
                cofactor(a, x.gcd(y), ap, gp)
                ap.gcd(gp) = Nat.1
                ap.coprime(gp)
                gcd_mult_left(ap, gp * xp, a.gcd(x.gcd(y)))
                a.gcd(x.gcd(y)) * ap.gcd(gp * xp) = (a.gcd(x.gcd(y)) * ap).gcd(a.gcd(x.gcd(y)) * (gp * xp))
                a.gcd(x.gcd(y)) * ap = a
                a.gcd(x.gcd(y)) * (gp * xp) = x.gcd(y) * xp
                x.gcd(y) * xp = x
                (a.gcd(x.gcd(y)) * ap).gcd(a.gcd(x.gcd(y)) * (gp * xp)) = a.gcd(x)
                a.gcd(x.gcd(y)) * ap.gcd(gp * xp) = a.gcd(x)
                gcd_coprime_factor(ap, gp, xp)
                ap.gcd(gp * xp) = ap.gcd(xp)
                a.gcd(x.gcd(y)) * ap.gcd(xp) = a.gcd(x)
                gcd_mult_left(ap, gp * yp, a.gcd(x.gcd(y)))
                a.gcd(x.gcd(y)) * ap.gcd(gp * yp) = (a.gcd(x.gcd(y)) * ap).gcd(a.gcd(x.gcd(y)) * (gp * yp))
                a.gcd(x.gcd(y)) * (gp * yp) = x.gcd(y) * yp
                x.gcd(y) * yp = y
                (a.gcd(x.gcd(y)) * ap).gcd(a.gcd(x.gcd(y)) * (gp * yp)) = a.gcd(y)
                a.gcd(x.gcd(y)) * ap.gcd(gp * yp) = a.gcd(y)
                gcd_coprime_factor(ap, gp, yp)
                ap.gcd(gp * yp) = ap.gcd(yp)
                a.gcd(x.gcd(y)) * ap.gcd(yp) = a.gcd(y)
                gcd_mult_left(ap, gp * (xp * yp), a.gcd(x.gcd(y)))
                a.gcd(x.gcd(y)) * ap.gcd(gp * (xp * yp)) = (a.gcd(x.gcd(y)) * ap).gcd(a.gcd(x.gcd(y)) * (gp * (xp * yp)))
                a.gcd(x.gcd(y)) * (gp * (xp * yp)) = x.gcd(y) * (xp * yp)
                x.gcd(y) * xp * yp = x.lcm(y)
                x.gcd(y) * (xp * yp) = x.gcd(y) * xp * yp
                a.gcd(x.gcd(y)) * (gp * (xp * yp)) = x.lcm(y)
                (a.gcd(x.gcd(y)) * ap).gcd(a.gcd(x.gcd(y)) * (gp * (xp * yp))) = a.gcd(x.lcm(y))
                a.gcd(x.gcd(y)) * ap.gcd(gp * (xp * yp)) = a.gcd(x.lcm(y))
                gcd_coprime_factor(ap, gp, xp * yp)
                ap.gcd(gp * (xp * yp)) = ap.gcd(xp * yp)
                a.gcd(x.gcd(y)) * ap.gcd(xp * yp) = a.gcd(x.lcm(y))
                gcd_mul_coprime(ap, xp, yp)
                ap.gcd(xp * yp) = ap.gcd(xp) * ap.gcd(yp)
                a.gcd(x.gcd(y)) * (ap.gcd(xp) * ap.gcd(yp)) = a.gcd(x.lcm(y))
                a.gcd(x) * a.gcd(y) = (a.gcd(x.gcd(y)) * ap.gcd(xp)) * (a.gcd(x.gcd(y)) * ap.gcd(yp))
                (a.gcd(x.gcd(y)) * ap.gcd(xp)) * (a.gcd(x.gcd(y)) * ap.gcd(yp)) = a.gcd(x.gcd(y)) * (ap.gcd(xp) * (a.gcd(x.gcd(y)) * ap.gcd(yp)))
                ap.gcd(xp) * (a.gcd(x.gcd(y)) * ap.gcd(yp)) = (ap.gcd(xp) * a.gcd(x.gcd(y))) * ap.gcd(yp)
                ap.gcd(xp) * a.gcd(x.gcd(y)) = a.gcd(x.gcd(y)) * ap.gcd(xp)
                (ap.gcd(xp) * a.gcd(x.gcd(y))) * ap.gcd(yp) = (a.gcd(x.gcd(y)) * ap.gcd(xp)) * ap.gcd(yp)
                ap.gcd(xp) * (a.gcd(x.gcd(y)) * ap.gcd(yp)) = (a.gcd(x.gcd(y)) * ap.gcd(xp)) * ap.gcd(yp)
                a.gcd(x.gcd(y)) * (ap.gcd(xp) * (a.gcd(x.gcd(y)) * ap.gcd(yp))) = a.gcd(x.gcd(y)) * ((a.gcd(x.gcd(y)) * ap.gcd(xp)) * ap.gcd(yp))
                a.gcd(x.gcd(y)) * ((a.gcd(x.gcd(y)) * ap.gcd(xp)) * ap.gcd(yp)) = a.gcd(x.gcd(y)) * (a.gcd(x.gcd(y)) * (ap.gcd(xp) * ap.gcd(yp)))
                a.gcd(x.gcd(y)) * (a.gcd(x.gcd(y)) * (ap.gcd(xp) * ap.gcd(yp))) = a.gcd(x.gcd(y)) * a.gcd(x.lcm(y))
                a.gcd(x) * a.gcd(y) = a.gcd(x.gcd(y)) * a.gcd(x.lcm(y))
            }
        }
    }
}

/// The gcd of a with the lcm of b and c divides the lcm of the two gcds.
theorem gcd_lcm_distrib_right(a: Nat, b: Nat, c: Nat) {
    a.gcd(b.lcm(c)).divides((a.gcd(b)).lcm(a.gcd(c)))
} by {
    gcd_lcm_identity(a, b, c)
    a.gcd(b) * a.gcd(c) = a.gcd(b.gcd(c)) * a.gcd(b.lcm(c))
    gcd_assoc(a, b, c)
    (a.gcd(b)).gcd(a.gcd(c)) = a.gcd(b.gcd(c))
    a.gcd(b.gcd(c)) * a.gcd(b.lcm(c)) = (a.gcd(b)).gcd(a.gcd(c)) * a.gcd(b.lcm(c))
    a.gcd(b) * a.gcd(c) = (a.gcd(b)).gcd(a.gcd(c)) * a.gcd(b.lcm(c))
    gcd_mul_lcm(a.gcd(b), a.gcd(c))
    (a.gcd(b)).gcd(a.gcd(c)) * (a.gcd(b)).lcm(a.gcd(c)) = a.gcd(b) * a.gcd(c)
    (a.gcd(b)).gcd(a.gcd(c)) * a.gcd(b.lcm(c)) = (a.gcd(b)).gcd(a.gcd(c)) * (a.gcd(b)).lcm(a.gcd(c))
    if (a.gcd(b)).gcd(a.gcd(c)) = Nat.0 {
        gcd_nonzero_left(a.gcd(b), a.gcd(c))
        a.gcd(b) = Nat.0
        gcd_nonzero_left(a, b)
        a = Nat.0
        gcd_zero_left(b)
        a.gcd(b) = b
        b = Nat.0
        gcd_zero_left(c)
        a.gcd(c) = c
        c = Nat.0
        lcm_zero_left(c)
        b.lcm(c) = Nat.0
        gcd_zero_left(b.lcm(c))
        a.gcd(b.lcm(c)) = b.lcm(c)
        b.lcm(c) = Nat.0
        a.gcd(b.lcm(c)) = Nat.0
        a.gcd(b) = Nat.0
        gcd_zero_left(a.gcd(b))
        (a.gcd(b)).gcd(a.gcd(c)) = a.gcd(c)
        a.gcd(c) = Nat.0
        (a.gcd(b)).gcd(a.gcd(c)) = Nat.0
        lcm_zero_left(a.gcd(c))
        (a.gcd(b)).lcm(a.gcd(c)) = Nat.0
        a.gcd(b.lcm(c)).divides((a.gcd(b)).lcm(a.gcd(c)))
    } else {
        (a.gcd(b)).gcd(a.gcd(c)) != Nat.0
        mul_cancel_left((a.gcd(b)).gcd(a.gcd(c)), a.gcd(b.lcm(c)), (a.gcd(b)).lcm(a.gcd(c)))
        a.gcd(b.lcm(c)) = (a.gcd(b)).lcm(a.gcd(c))
        divides_self(a.gcd(b.lcm(c)))
        a.gcd(b.lcm(c)).divides(a.gcd(b.lcm(c)))
        a.gcd(b.lcm(c)).divides((a.gcd(b)).lcm(a.gcd(c)))
    }
}

/// The gcd distributes over the lcm:
/// gcd(a, lcm(b, c)) = lcm(gcd(a, b), gcd(a, c)).
theorem gcd_lcm_distrib(a: Nat, b: Nat, c: Nat) {
    a.gcd(b.lcm(c)) = (a.gcd(b)).lcm(a.gcd(c))
} by {
    gcd_lcm_distrib_left(a, b, c)
    (a.gcd(b)).lcm(a.gcd(c)).divides(a.gcd(b.lcm(c)))
    gcd_lcm_distrib_right(a, b, c)
    a.gcd(b.lcm(c)).divides((a.gcd(b)).lcm(a.gcd(c)))
    divides_symm(a.gcd(b.lcm(c)), (a.gcd(b)).lcm(a.gcd(c)))
    a.gcd(b.lcm(c)) = (a.gcd(b)).lcm(a.gcd(c))
}
