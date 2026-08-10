// The prime-factor-counting functions Omega(n) (number of prime factors with
// multiplicity) and omega(n) (number of distinct prime factors), together with
// their basic arithmetic properties: prime-power values, additivity over
// products, and the connections to Liouville's lambda and the Mobius function.
//
// The function Omega is `nat_prime_omega` from liouville.ac (the length of the
// canonical prime factorisation); this file reuses it and adds omega as the
// length of the factorisation with duplicates removed.
from nat import Nat, mul_to_zero, exp_ne_zero, lt_suc, lt_and_lte, lt_not_ref,
    lte_ref, lt_imp_lte_suc, lte_imp_not_lt
from int import Int
from list import List, is_permutation, permutation_preserves_length,
    unique_length, unique_preserves_contains, unique_list_is_unique,
    unique_same_contains_imp_permutation, list_contains_implies_count_geq_one,
    list_not_contains_impl_count_zero, not_contains_add, add_contains_left,
    add_contains_right, add_contains_or, add_length, unique_list_sum,
    cons_unique_of_tail_unique_not_contains, product, product_append
from algebra.ring.ring import Ring, alternating_sign, alternating_sign_eq_neg_one_pow
from number_theory.factorisation import prime_factorisation, prime_factorisation_product,
    prime_factorisation_all_prime, prime_factorisation_unique, all_prime,
    all_prime_append, all_prime_only_primes, coprime_imp_no_shared_prime_factor
from number_theory.liouville import nat_prime_omega, nat_liouville, nat_prime_omega_mul
from number_theory.mobius_inversion import nat_mobius, prime_factorisation_prime,
    prime_factorisation_one, nat_mobius_nonzero_iff_factorisation_unique,
    list_member_divides_product
numerals Nat
numerals Int

/// The number of distinct prime factors of `n` (the function `omega(n)`): the
/// length of the prime factorisation with duplicate entries removed. For zero
/// the empty placeholder factorisation makes this zero.
define nat_omega(n: Nat) -> Nat {
    prime_factorisation(n).unique.length
}

/// Step case of the squarefree-length characterisation: if the tail is unique
/// whenever its distinct entries match its entries, then the same holds for the
/// cons list.
theorem unique_length_eq_imp_is_unique_cons[T](head: T, tail: List[T]) {
    (tail.unique.length = tail.length implies tail.is_unique) implies
        (List.cons(head, tail).unique.length = List.cons(head, tail).length implies
            List.cons(head, tail).is_unique)
} by {
    if tail.unique.length = tail.length implies tail.is_unique {
        if List.cons(head, tail).unique.length = List.cons(head, tail).length {
            if tail.contains(head) {
                List.cons(head, tail).unique = tail.unique
                List.cons(head, tail).length = tail.length.suc
                tail.unique.length = tail.length.suc
                unique_length(tail)
                tail.unique.length <= tail.length
                tail.length.suc <= tail.length
                lt_suc(tail.length)
                tail.length < tail.length.suc
                lt_and_lte(tail.length, tail.length.suc, tail.length)
                tail.length < tail.length
                lt_not_ref(tail.length)
                false
            }
            List.cons(head, tail).unique = List.cons(head, tail.unique)
            List.cons(head, tail).unique.length = tail.unique.length.suc
            List.cons(head, tail).length = tail.length.suc
            tail.unique.length.suc = tail.length.suc
            tail.unique.length = tail.length
            tail.is_unique
            cons_unique_of_tail_unique_not_contains(head, tail)
            List.cons(head, tail).is_unique
        }
    }
}

/// A list whose distinct entries are as many as its entries has no repeated
/// entries.
theorem unique_length_eq_imp_is_unique[T](list: List[T]) {
    list.unique.length = list.length implies list.is_unique
} by {
    define p(l: List[T]) -> Bool {
        l.unique.length = l.length implies l.is_unique
    }
    List.nil[T].unique = List.nil[T]
    List.nil[T].unique.length = Nat.0
    List.nil[T].length = Nat.0
    p(List.nil[T])
    forall(head: T, tail: List[T]) {
        if p(tail) {
            unique_length_eq_imp_is_unique_cons(head, tail)
            p(List.cons(head, tail)) =
                (List.cons(head, tail).unique.length = List.cons(head, tail).length implies
                    List.cons(head, tail).is_unique)
            p(List.cons(head, tail))
        }
    }
    forall(head: T, tail: List[T]) {
        p(tail) implies p(List.cons(head, tail))
    }
    p(List.nil[T]) and forall(head: T, tail: List[T]) {
        p(tail) implies p(List.cons(head, tail))
    }
    List.induction(function(l: List[T]) { p(l) })
    forall(l: List[T]) { p(l) }
    p(list)
}

/// A list has exactly as many distinct entries as entries exactly when it has
/// no repeated entries.
theorem unique_length_eq_iff_is_unique[T](list: List[T]) {
    (list.unique.length = list.length) = list.is_unique
} by {
    if list.is_unique {
        list.unique = list
        list.unique.length = list.length
    }
    if list.unique.length = list.length {
        unique_length_eq_imp_is_unique(list)
        list.is_unique
    }
}

/// Permutations preserve the number of distinct entries.
theorem permutation_preserves_unique_length[T](a: List[T], b: List[T]) {
    is_permutation(a, b) implies a.unique.length = b.unique.length
} by {
    if is_permutation(a, b) {
        is_permutation(a, b) = forall(x: T) { a.count(x) = b.count(x) }
        forall(x: T) {
            if a.contains(x) {
                list_contains_implies_count_geq_one(a, x)
                a.count(x) >= Nat.1
                a.count(x) = b.count(x)
                b.count(x) >= Nat.1
                if not b.contains(x) {
                    list_not_contains_impl_count_zero(b, x)
                    b.count(x) = Nat.0
                    Nat.1 <= Nat.0
                    false
                }
                b.contains(x)
            }
            if b.contains(x) {
                list_contains_implies_count_geq_one(b, x)
                b.count(x) >= Nat.1
                a.count(x) = b.count(x)
                a.count(x) >= Nat.1
                if not a.contains(x) {
                    list_not_contains_impl_count_zero(a, x)
                    a.count(x) = Nat.0
                    Nat.1 <= Nat.0
                    false
                }
                a.contains(x)
            }
            a.contains(x) = b.contains(x)
        }
        forall(x: T) {
            unique_preserves_contains(a, x)
            unique_preserves_contains(b, x)
            a.unique.contains(x) = a.contains(x)
            b.unique.contains(x) = b.contains(x)
            a.contains(x) = b.contains(x)
            a.unique.contains(x) = b.unique.contains(x)
        }
        unique_list_is_unique(a)
        unique_list_is_unique(b)
        a.unique.is_unique
        b.unique.is_unique
        unique_same_contains_imp_permutation(a.unique, b.unique)
        is_permutation(a.unique, b.unique)
        permutation_preserves_length(a.unique, b.unique)
        a.unique.length = b.unique.length
    }
}

/// The total number of prime factors of a prime is one: `Omega(p) = 1`.
theorem nat_prime_omega_prime(p: Nat) {
    p.is_prime implies nat_prime_omega(p) = Nat.1
} by {
    if p.is_prime {
        prime_factorisation_prime(p)
        prime_factorisation(p) = List.singleton(p)
        List.singleton(p).length = Nat.1
        nat_prime_omega(p) = prime_factorisation(p).length
        nat_prime_omega(p) = Nat.1
    }
}

/// The number of distinct prime factors of a prime is one: `omega(p) = 1`.
theorem nat_omega_prime(p: Nat) {
    p.is_prime implies nat_omega(p) = Nat.1
} by {
    if p.is_prime {
        prime_factorisation_prime(p)
        prime_factorisation(p) = List.singleton(p)
        List.singleton(p).unique = List.singleton(p)
        List.singleton(p).unique.length = List.singleton(p).length
        List.singleton(p).length = Nat.1
        List.singleton(p).unique.length = Nat.1
        nat_omega(p) = prime_factorisation(p).unique.length
        nat_omega(p) = Nat.1
    }
}

/// The total number of prime factors of a prime power: `Omega(p^k) = k` for a
/// prime `p`, in particular for every `k >= 1`.
theorem nat_prime_omega_pow(p: Nat, k: Nat) {
    p.is_prime implies nat_prime_omega(p.pow(k)) = k
} by {
    if p.is_prime {
        let f: Nat -> Bool = function(x: Nat) {
            nat_prime_omega(p.pow(x)) = x
        }
        p.pow(Nat.0) = Nat.1
        prime_factorisation_one
        prime_factorisation(Nat.1) = List.nil[Nat]
        List.nil[Nat].length = Nat.0
        nat_prime_omega(Nat.1) = prime_factorisation(Nat.1).length
        nat_prime_omega(Nat.1) = Nat.0
        nat_prime_omega(p.pow(Nat.0)) = Nat.0
        f(Nat.0)
        forall(x: Nat) {
            if f(x) {
                nat_prime_omega(p.pow(x)) = x
                Nat.1 < p
                p != Nat.0
                exp_ne_zero(p, x)
                p.pow(x) != Nat.0
                p.pow(x.suc) = p * p.pow(x)
                nat_prime_omega_mul(p, p.pow(x))
                nat_prime_omega(p * p.pow(x)) = nat_prime_omega(p) + nat_prime_omega(p.pow(x))
                nat_prime_omega_prime(p)
                nat_prime_omega(p) = Nat.1
                nat_prime_omega(p.pow(x)) = x
                nat_prime_omega(p.pow(x.suc)) = Nat.1 + x
                Nat.1 + x = x.suc
                nat_prime_omega(p.pow(x.suc)) = x.suc
                f(x.suc)
            }
        }
        forall(x: Nat) {
            f(x) implies f(x.suc)
        }
        f(Nat.0) and forall(x: Nat) {
            f(x) implies f(x.suc)
        }
        Nat.induction(f)
        forall(x: Nat) { f(x) }
        f(k)
        nat_prime_omega(p.pow(k)) = k
    }
}

/// The total number of prime factors of a product of positive naturals is the
/// sum of the numbers for the factors: `Omega(mn) = Omega(m) + Omega(n)` for
/// every `m, n > 0` (no coprimality is needed, since the count is with
/// multiplicity).
theorem nat_big_omega_mul(a: Nat, b: Nat) {
    a != Nat.0 and b != Nat.0 implies
        nat_prime_omega(a * b) = nat_prime_omega(a) + nat_prime_omega(b)
} by {
    if a != Nat.0 and b != Nat.0 {
        nat_prime_omega_mul(a, b)
        nat_prime_omega(a * b) = nat_prime_omega(a) + nat_prime_omega(b)
    }
}

/// The number of distinct prime factors of a product of coprime positive
/// naturals is the sum of the numbers for the factors: `omega(mn) =
/// omega(m) + omega(n)` for coprime `m, n > 0`.
theorem nat_omega_mul(a: Nat, b: Nat) {
    a != Nat.0 and b != Nat.0 and a.coprime(b) implies
        nat_omega(a * b) = nat_omega(a) + nat_omega(b)
} by {
    if a != Nat.0 and b != Nat.0 and a.coprime(b) {
        Nat.1 <= a
        Nat.1 <= b
        let fa: List[Nat] = prime_factorisation(a)
        let fb: List[Nat] = prime_factorisation(b)
        prime_factorisation_product(a)
        prime_factorisation_product(b)
        prime_factorisation_all_prime(a)
        prime_factorisation_all_prime(b)
        all_prime(fa)
        all_prime(fb)
        product[Nat](fa) = a
        product[Nat](fb) = b
        let combined: List[Nat] = fa + fb
        all_prime_append(fa, fb)
        all_prime(combined)
        product_append[Nat](fa, fb)
        product[Nat](combined) = product[Nat](fa) * product[Nat](fb)
        product[Nat](combined) = a * b
        if a * b = Nat.0 {
            mul_to_zero(a, b)
            a = Nat.0 or b = Nat.0
            false
        }
        a * b != Nat.0
        Nat.1 <= a * b
        prime_factorisation_product(a * b)
        prime_factorisation_all_prime(a * b)
        all_prime(prime_factorisation(a * b))
        product[Nat](prime_factorisation(a * b)) = a * b
        prime_factorisation_unique(combined, prime_factorisation(a * b))
        is_permutation(combined, prime_factorisation(a * b))
        permutation_preserves_unique_length(combined, prime_factorisation(a * b))
        combined.unique.length = prime_factorisation(a * b).unique.length
        forall(x: Nat) {
            if fa.contains(x) {
                all_prime_only_primes(fa, x)
                x.is_prime
                list_member_divides_product(fa, x)
                x.divides(product[Nat](fa))
                x.divides(a)
                if fb.contains(x) {
                    all_prime_only_primes(fb, x)
                    x.is_prime
                    list_member_divides_product(fb, x)
                    x.divides(product[Nat](fb))
                    x.divides(b)
                    coprime_imp_no_shared_prime_factor(a, b, x)
                    not (x.divides(a) and x.divides(b))
                    x.divides(a) and x.divides(b)
                    false
                }
                not fb.contains(x)
            }
        }
        forall(x: Nat) { fa.contains(x) implies not fb.contains(x) }
        // The distinct entries of the combined factorisation are exactly the
        // distinct entries of the two deduplicated factorisations, since the
        // factorisations share no prime.
        forall(x: Nat) {
            if combined.unique.contains(x) {
                unique_preserves_contains(combined, x)
                combined.contains(x)
                add_contains_or(fa, fb, x)
                fa.contains(x) or fb.contains(x)
                if fa.contains(x) {
                    unique_preserves_contains(fa, x)
                    fa.unique.contains(x)
                    add_contains_left(fa.unique, fb.unique, x)
                    (fa.unique + fb.unique).contains(x)
                } else {
                    fb.contains(x)
                    unique_preserves_contains(fb, x)
                    fb.unique.contains(x)
                    add_contains_right(fa.unique, fb.unique, x)
                    (fa.unique + fb.unique).contains(x)
                }
                (fa.unique + fb.unique).contains(x)
                unique_preserves_contains(fa.unique + fb.unique, x)
                (fa.unique + fb.unique).unique.contains(x)
            }
            if (fa.unique + fb.unique).unique.contains(x) {
                unique_preserves_contains(fa.unique + fb.unique, x)
                (fa.unique + fb.unique).contains(x)
                add_contains_or(fa.unique, fb.unique, x)
                fa.unique.contains(x) or fb.unique.contains(x)
                if fa.unique.contains(x) {
                    unique_preserves_contains(fa, x)
                    fa.contains(x)
                    add_contains_left(fa, fb, x)
                    (fa + fb).contains(x)
                } else {
                    fb.unique.contains(x)
                    unique_preserves_contains(fb, x)
                    fb.contains(x)
                    add_contains_right(fa, fb, x)
                    (fa + fb).contains(x)
                }
                (fa + fb).contains(x)
                combined.contains(x)
                unique_preserves_contains(combined, x)
                combined.unique.contains(x)
            }
            combined.unique.contains(x) = (fa.unique + fb.unique).unique.contains(x)
        }
        forall(x: Nat) {
            combined.unique.contains(x) = (fa.unique + fb.unique).unique.contains(x)
        }
        unique_list_is_unique(combined)
        unique_list_is_unique(fa.unique + fb.unique)
        combined.unique.is_unique
        (fa.unique + fb.unique).unique.is_unique
        unique_same_contains_imp_permutation(combined.unique, (fa.unique + fb.unique).unique)
        is_permutation(combined.unique, (fa.unique + fb.unique).unique)
        permutation_preserves_length(combined.unique, (fa.unique + fb.unique).unique)
        combined.unique.length = (fa.unique + fb.unique).unique.length
        // The deduplicated factorisations are disjoint, so their distinct
        // counts add.
        forall(x: Nat) {
            if fa.unique.contains(x) and fb.unique.contains(x) {
                fa.unique.contains(x)
                unique_preserves_contains(fa, x)
                fa.contains(x)
                not fb.contains(x)
                false
            }
            not (fa.unique.contains(x) and fb.unique.contains(x))
        }
        forall(x: Nat) { not (fa.unique.contains(x) and fb.unique.contains(x)) }
        unique_list_sum(fa.unique, fb.unique)
        (fa.unique + fb.unique).is_unique
        (fa.unique + fb.unique).unique = fa.unique + fb.unique
        (fa.unique + fb.unique).unique.length = (fa.unique + fb.unique).length
        add_length(fa.unique, fb.unique)
        (fa.unique + fb.unique).length = fa.unique.length + fb.unique.length
        (fa.unique + fb.unique).unique.length = fa.unique.length + fb.unique.length
        combined.unique.length = fa.unique.length + fb.unique.length
        prime_factorisation(a * b).unique.length = fa.unique.length + fb.unique.length
        nat_omega(a * b) = prime_factorisation(a * b).unique.length
        nat_omega(a * b) = fa.unique.length + fb.unique.length
        nat_omega(a) = prime_factorisation(a).unique.length
        nat_omega(b) = prime_factorisation(b).unique.length
        nat_omega(a * b) = nat_omega(a) + nat_omega(b)
    }
}

/// The Liouville function is minus one to the power of the number of prime
/// factors with multiplicity: `lambda(n) = (-1)^Omega(n)`.
theorem nat_liouville_eq_neg_one_pow_omega(n: Nat) {
    nat_liouville(n) = (-Int.1).pow(nat_prime_omega(n))
} by {
    alternating_sign_eq_neg_one_pow[Int](nat_prime_omega(n))
    alternating_sign[Int](nat_prime_omega(n)) = (-Int.1).pow(nat_prime_omega(n))
    nat_liouville(n) = alternating_sign[Int](nat_prime_omega(n))
    nat_liouville(n) = (-Int.1).pow(nat_prime_omega(n))
}

/// A natural has as many distinct prime factors as prime factors with
/// multiplicity exactly when its prime factorisation has no repeated primes
/// (equivalently, `n` is squarefree).
theorem nat_omega_eq_Omega_iff_factorisation_unique(n: Nat) {
    (nat_omega(n) = nat_prime_omega(n)) = prime_factorisation(n).is_unique
} by {
    unique_length_eq_iff_is_unique(prime_factorisation(n))
    (prime_factorisation(n).unique.length = prime_factorisation(n).length) =
        prime_factorisation(n).is_unique
    nat_omega(n) = prime_factorisation(n).unique.length
    nat_prime_omega(n) = prime_factorisation(n).length
    (nat_omega(n) = nat_prime_omega(n)) = prime_factorisation(n).is_unique
}

/// A positive natural is squarefree exactly when the number of its distinct
/// prime factors equals the number of its prime factors with multiplicity;
/// equivalently, the Mobius function does not vanish there.
theorem nat_mobius_nonzero_iff_omega_eq_Omega(n: Nat) {
    Nat.0 < n implies ((nat_mobius(n) != Int.0) = (nat_omega(n) = nat_prime_omega(n)))
} by {
    if Nat.0 < n {
        nat_mobius_nonzero_iff_factorisation_unique(n)
        (nat_mobius(n) != Int.0) = prime_factorisation(n).is_unique
        nat_omega_eq_Omega_iff_factorisation_unique(n)
        (nat_omega(n) = nat_prime_omega(n)) = prime_factorisation(n).is_unique
        (nat_mobius(n) != Int.0) = (nat_omega(n) = nat_prime_omega(n))
    }
}
