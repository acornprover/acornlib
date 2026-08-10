/// Arithmetic-function generating identities.
///
/// This file records the bridge between the arithmetic functions of
/// `number_theory` — sigma, tau, Euler's totient, and the Möbius function —
/// and their Dirichlet-series / generating-function identities.  The
/// arithmetic content is expressed through the divisor-sum operator
/// `divisor_sum_fn(f)(n) = sum_{d | n} f(d)`: the defining divisor-sum of
/// sigma, the classical convolution identity
/// `sum_{d | n} sigma(d) = sum_{d | n} d * tau(n / d)`, the summation
/// identity `sum_{d | n} totient(d) = n`, and the coefficient-level content
/// of the Euler products for the partition function and the Möbius function.
///
/// The analytic statements — the partition generating function
/// `sum_n p(n) x^n = product_k 1 / (1 - x^k)` and the Euler product
/// `product_p (1 - p^{-s}) = sum_n mu(n) / n^s` — are infinite-series
/// identities that need the infinite-product machinery the library does not
/// yet have (partial products and their limits, in the style of
/// `combinatorics.generating_functions`); they are recorded as comments in
/// the sections below, alongside the finite statements proved here.

from nat import Nat
from int import Int
from list import List, map, sum, sum_map_of_pointwise
from number_theory.divisor_sum import divisor_list, divisor_sum_fn, divisor_sum_fn_apply,
    nat_sigma, nat_tau, divisor_sum_fn_nat_identity_arithmetic_fn_eq_sigma
from number_theory.arithmetic_functions import nat_identity_arithmetic_fn,
    nat_identity_arithmetic_fn_apply, nat_dirichlet_unit_fn, nat_dirichlet_unit_fn_at_one,
    nat_dirichlet_unit_fn_off_one
from number_theory.totient import nat_totient
from number_theory.totient_sums import totient_divisor_sum_identity, divisor_quotient_reindex
from number_theory.dirichlet import divisor_quotient, dirichlet_term, dirichlet_term_apply
from number_theory.divisor_identities import nat_sigma_divisor_sum_eq_id_convolve_tau_sum
from number_theory.mobius_inversion import nat_mobius, nat_mobius_divisor_sum
numerals Nat
numerals Int

// ---------------------------------------------------------------------------
// Sigma as a divisor sum:  sum_{d | n} d = sigma(n).
//
// The classical identity defining sigma is the divisor sum of the identity
// arithmetic function.  `divisor_sum_fn_nat_identity_arithmetic_fn_eq_sigma`
// in `number_theory.divisor_sum` proves `divisor_sum_fn(nat_identity_arithmetic_fn)(n) =
// nat_sigma(n)`; the restatement below puts `nat_sigma` on the left, the
// order used by the generating identities of this file.  The Dirichlet-series
// reading is `sum_n sigma(n) / n^s = zeta(s) * zeta(s - 1)`: the divisor sum
// is the Dirichlet convolution `id * 1`, whose Dirichlet series is the product
// of the series of `id` and `1`.
// ---------------------------------------------------------------------------

/// The divisor-sum form of sigma: `sum_{d | n} d = sigma(n)`.
theorem nat_sigma_divisor_sum_identity(n: Nat) {
    nat_sigma(n) = divisor_sum_fn(nat_identity_arithmetic_fn)(n)
} by {
    divisor_sum_fn_nat_identity_arithmetic_fn_eq_sigma(n)
    divisor_sum_fn(nat_identity_arithmetic_fn)(n) = nat_sigma(n)
}

/// The divisor sum of `sigma` is symmetric under the cofactor map
/// `d -> n / d`: `sum_{d | n} sigma(n / d) = sum_{d | n} sigma(d)` for
/// positive `n`.  The cofactor map permutes the divisors of `n`, so summing
/// `sigma` over the cofactor image gives the same value as summing it over
/// the divisor list itself.
theorem nat_sigma_divisor_sum_cofactor(n: Nat) {
    Nat.0 < n implies
        divisor_sum_fn(function(d: Nat) { nat_sigma(divisor_quotient(n, d)) })(n) =
            divisor_sum_fn(nat_sigma)(n)
} by {
    if Nat.0 < n {
        divisor_sum_fn_apply(function(d: Nat) { nat_sigma(divisor_quotient(n, d)) }, n)
        divisor_sum_fn(function(d: Nat) { nat_sigma(divisor_quotient(n, d)) })(n) =
            sum(map(divisor_list(n), function(d: Nat) { nat_sigma(divisor_quotient(n, d)) }))
        divisor_quotient_reindex(nat_sigma, n)
        sum(map(divisor_list(n), function(d: Nat) { nat_sigma(divisor_quotient(n, d)) })) =
            sum(map(divisor_list(n), nat_sigma))
        divisor_sum_fn_apply(nat_sigma, n)
        divisor_sum_fn(nat_sigma)(n) = sum(map(divisor_list(n), nat_sigma))
        divisor_sum_fn(function(d: Nat) { nat_sigma(divisor_quotient(n, d)) })(n) =
            divisor_sum_fn(nat_sigma)(n)
    }
}

/// The classical convolution identity `sum_{d | n} sigma(d) =
/// sum_{d | n} d * tau(n / d)`, with the right-hand side unfolded from the
/// Dirichlet-convolution form `dirichlet_convolve(nat_identity_arithmetic_fn,
/// nat_tau)(n)` to the explicit divisor sum.  `divisor_identities.ac` proves
/// the convolution form; this theorem records the fully unfolded divisor-sum
/// statement, which is the arithmetic content of the Dirichlet-series identity
/// `sum_n sigma(n) / n^s = zeta(s) * zeta(s - 1)` at the coefficient level.
theorem nat_sigma_divisor_sum_id_convolve_tau_term(n: Nat) {
    divisor_sum_fn(nat_sigma)(n) =
        sum(map(divisor_list(n), function(d: Nat) {
            d * nat_tau(divisor_quotient(n, d))
        }))
} by {
    nat_sigma_divisor_sum_eq_id_convolve_tau_sum(n)
    divisor_sum_fn(nat_sigma)(n) =
        sum(map(divisor_list(n), dirichlet_term(nat_identity_arithmetic_fn, nat_tau, n)))
    forall(d: Nat) {
        if divisor_list(n).contains(d) {
            dirichlet_term_apply(nat_identity_arithmetic_fn, nat_tau, n, d)
            dirichlet_term(nat_identity_arithmetic_fn, nat_tau, n)(d) =
                nat_identity_arithmetic_fn(d) * nat_tau(divisor_quotient(n, d))
            nat_identity_arithmetic_fn_apply(d)
            nat_identity_arithmetic_fn(d) = d
            dirichlet_term(nat_identity_arithmetic_fn, nat_tau, n)(d) =
                d * nat_tau(divisor_quotient(n, d))
        }
    }
    sum_map_of_pointwise(divisor_list(n),
        dirichlet_term(nat_identity_arithmetic_fn, nat_tau, n),
        function(d: Nat) { d * nat_tau(divisor_quotient(n, d)) })
    sum(map(divisor_list(n), dirichlet_term(nat_identity_arithmetic_fn, nat_tau, n))) =
        sum(map(divisor_list(n), function(d: Nat) {
            d * nat_tau(divisor_quotient(n, d))
        }))
    divisor_sum_fn(nat_sigma)(n) =
        sum(map(divisor_list(n), function(d: Nat) {
            d * nat_tau(divisor_quotient(n, d))
        }))
}

// ---------------------------------------------------------------------------
// The generating function of the partition function.
//
// For |x| < 1, the generating function of the partition function `p` is the
// infinite product of geometric series
//
//     sum_{n >= 0} p(n) x^n  =  product_{k >= 1} 1 / (1 - x^k),
//
// where `p(n)` is the number of partitions of `n` (the function `p` of
// `combinatorics.partitions`).  Expanding the product, the coefficient of
// `x^n` counts, for each part size `k`, how many copies of `x^k` are chosen,
// which is exactly the number of ways to write `n` as a sum of positive
// integers — the combinatorial content of the identity.
//
// The identity is recorded, but not yet proved, in `combinatorics.partitions`
// (its closing section on the classic identities).  A proof needs the
// infinite-product machinery — the sequence of partial products
// `product_{k <= N} 1 / (1 - x^k)` and its limit, in the style of
// `combinatorics.generating_functions` — together with the set-cardinality
// form of `p`.  It is therefore stated here as a comment rather than as a
// theorem, per the library convention of recording unproved statements.
// ---------------------------------------------------------------------------

// ---------------------------------------------------------------------------
// The Euler product for the Möbius function.
//
// The Dirichlet series of the Möbius function is the reciprocal of the zeta
// function, with the Euler product over the primes
//
//     sum_{n >= 1} mu(n) / n^s  =  product_{p prime} (1 - 1 / p^s)
//                                =  1 / zeta(s).
//
// The finite (coefficient-level) content of this identity is the fundamental
// identity of the Möbius function, proved in `number_theory.mobius_inversion`
// as `nat_mobius_divisor_sum`: the divisor sum `sum_{d | n} mu(d)` is 1 at
// `n = 1` and 0 elsewhere.  In Dirichlet-convolution terms this is
// `mu * 1 = epsilon`, with `epsilon` the Dirichlet unit — exactly the
// arithmetic-function statement whose Dirichlet-series reading is
// `1 / zeta(s) * zeta(s) = 1`.  The restatement below phrases the
// fundamental identity in this unit form.
// ---------------------------------------------------------------------------

/// The fundamental identity of the Möbius function in Dirichlet-unit form:
/// the divisor sum of `mu` is the Dirichlet unit `epsilon`, i.e.
/// `mu * 1 = epsilon` coefficientwise.  This is the arithmetic content of the
/// Euler product `product_{p prime} (1 - 1 / p^s) = 1 / zeta(s)`: expanding
/// the product, the coefficient of `1 / n^s` is the divisor sum of `mu`.
theorem nat_mobius_divisor_sum_unit_form(n: Nat) {
    sum(map(divisor_list(n), nat_mobius)) = Int.from_nat(nat_dirichlet_unit_fn(n))
} by {
    nat_mobius_divisor_sum(n)
    sum(map(divisor_list(n), nat_mobius)) = if n = Nat.1 { Int.1 } else { Int.0 }
    if n = Nat.1 {
        nat_dirichlet_unit_fn_at_one
        nat_dirichlet_unit_fn(Nat.1) = Nat.1
        Int.from_nat(nat_dirichlet_unit_fn(n)) = Int.from_nat(Nat.1)
        Int.from_nat(Nat.1) = Int.1
        sum(map(divisor_list(n), nat_mobius)) = Int.from_nat(nat_dirichlet_unit_fn(n))
    } else {
        nat_dirichlet_unit_fn_off_one(n)
        nat_dirichlet_unit_fn(n) = Nat.0
        Int.from_nat(nat_dirichlet_unit_fn(n)) = Int.from_nat(Nat.0)
        Int.from_nat(Nat.0) = Int.0
        sum(map(divisor_list(n), nat_mobius)) = Int.from_nat(nat_dirichlet_unit_fn(n))
    }
}

// ---------------------------------------------------------------------------
// Euler's totient as a divisor sum:  sum_{d | n} totient(d) = n.
//
// The classical summation identity of the totient.  `totient_sums.ac` proves
// `divisor_sum_fn(nat_totient)(n) = n` by partitioning `[1, n]` into the
// fibers of `x -> gcd(x, n)`; the restatement below keeps the same shape for
// this file, and the corollary records the cofactor-symmetric form
// `sum_{d | n} totient(n / d) = n`.  The Dirichlet-series reading is
// `sum_n totient(n) / n^s = zeta(s - 1) / zeta(s)`.
// ---------------------------------------------------------------------------

/// The classical summation identity `sum_{d | n} totient(d) = n`.
theorem nat_totient_divisor_sum_identity(n: Nat) {
    divisor_sum_fn(nat_totient)(n) = n
} by {
    totient_divisor_sum_identity(n)
}

/// The cofactor form of the summation identity: `sum_{d | n} totient(n / d)
/// = n` for positive `n`.  The cofactor map `d -> n / d` permutes the
/// divisors of `n`, so summing the totient over the cofactor image of the
/// divisor list gives the same value as summing it over the divisor list
/// itself.
theorem nat_totient_divisor_sum_cofactor(n: Nat) {
    Nat.0 < n implies
        divisor_sum_fn(function(d: Nat) { nat_totient(divisor_quotient(n, d)) })(n) = n
} by {
    if Nat.0 < n {
        divisor_sum_fn_apply(function(d: Nat) { nat_totient(divisor_quotient(n, d)) }, n)
        divisor_sum_fn(function(d: Nat) { nat_totient(divisor_quotient(n, d)) })(n) =
            sum(map(divisor_list(n), function(d: Nat) { nat_totient(divisor_quotient(n, d)) }))
        divisor_quotient_reindex(nat_totient, n)
        sum(map(divisor_list(n), function(d: Nat) { nat_totient(divisor_quotient(n, d)) })) =
            sum(map(divisor_list(n), nat_totient))
        divisor_sum_fn_apply(nat_totient, n)
        divisor_sum_fn(nat_totient)(n) = sum(map(divisor_list(n), nat_totient))
        divisor_sum_fn(function(d: Nat) { nat_totient(divisor_quotient(n, d)) })(n) =
            divisor_sum_fn(nat_totient)(n)
        totient_divisor_sum_identity(n)
        divisor_sum_fn(nat_totient)(n) = n
        divisor_sum_fn(function(d: Nat) { nat_totient(divisor_quotient(n, d)) })(n) = n
    }
}
