from nat import Nat, double_addition_carry_count, digit_sum
from number_theory.factorisation import count_prime_factor,
    count_prime_factor_le_imp_divides, divides_imp_count_prime_factor_le,
    prime_pow_divides_iff
from number_theory.falling_product import central_binom, central_binom_ne_zero,
    falling_product, falling_product_nonzero, falling_product_prime_count_sum,
    count_prime_factor_falling_product, falling_product_prime_count_sum_zero,
    falling_product_prime_count_sum_one, falling_product_prime_count_sum_two,
    central_binom_two_adic_valuation, nat_two_prime
from number_theory.kummer_carry import central_binom_valuation_eq_double_addition_carry_count,
    double_addition_carry_count_eq_central_binom_valuation,
    double_addition_carry_count_two_eq_digit_sum,
    digit_sum_eq_double_addition_carry_count_two

numerals Nat

/// The falling-product valuation sum is bounded by the valuation of a target
/// natural at a prime.
define falling_product_target_prime_bound(p: Nat, n: Nat, k: Nat, target: Nat) -> Bool {
    falling_product_prime_count_sum(p, n, k) <= count_prime_factor(p, target)
}

/// Every prime valuation sum in the falling product is bounded by the matching
/// valuation of a target natural.
define falling_product_target_primewise_bound(n: Nat, k: Nat, target: Nat) -> Bool {
    forall(p: Nat) {
        p.is_prime implies falling_product_target_prime_bound(p, n, k, target)
    }
}

/// A primewise falling-product target bound gives the target bound at any
/// particular prime.
theorem falling_product_target_prime_bound_of_primewise_bound(
    p: Nat, n: Nat, k: Nat, target: Nat
) {
    p.is_prime and falling_product_target_primewise_bound(n, k, target)
        implies falling_product_target_prime_bound(p, n, k, target)
} by {
    if p.is_prime and falling_product_target_primewise_bound(n, k, target) {
        let h: Bool = p.is_prime implies falling_product_target_prime_bound(p, n, k, target)
        h
        falling_product_target_prime_bound(p, n, k, target)
    }
}

/// A falling-product target valuation bound for one prime bounds the matching
/// valuation of the falling product itself.
theorem count_prime_factor_falling_product_le_target_of_target_prime_bound(
    p: Nat, n: Nat, k: Nat, target: Nat
) {
    k < n and falling_product_target_prime_bound(p, n, k, target)
        implies count_prime_factor(p, falling_product(n, k)) <= count_prime_factor(p, target)
} by {
    if k < n and falling_product_target_prime_bound(p, n, k, target) {
        falling_product_prime_count_sum(p, n, k) <= count_prime_factor(p, target)
        count_prime_factor_falling_product(p, n, k)
        count_prime_factor(p, falling_product(n, k)) =
            falling_product_prime_count_sum(p, n, k)
        count_prime_factor(p, falling_product(n, k)) <= count_prime_factor(p, target)
    }
}

/// Primewise falling-product target bounds give the prime-count inequalities
/// needed for divisibility by the target.
theorem count_prime_factor_falling_product_le_target_of_target_primewise_bound(
    p: Nat, n: Nat, k: Nat, target: Nat
) {
    p.is_prime and k < n and falling_product_target_primewise_bound(n, k, target)
        implies count_prime_factor(p, falling_product(n, k)) <= count_prime_factor(p, target)
} by {
    if p.is_prime and k < n and falling_product_target_primewise_bound(n, k, target) {
        falling_product_target_prime_bound_of_primewise_bound(p, n, k, target)
        falling_product_target_prime_bound(p, n, k, target)
        count_prime_factor_falling_product_le_target_of_target_prime_bound(p, n, k, target)
        count_prime_factor(p, falling_product(n, k)) <= count_prime_factor(p, target)
    }
}

/// Primewise target valuation bounds imply divisibility of the target by the
/// falling product.
theorem falling_product_divides_target_of_target_primewise_bound(
    n: Nat, k: Nat, target: Nat
) {
    k < n and target != Nat.0 and falling_product_target_primewise_bound(n, k, target)
        implies falling_product(n, k).divides(target)
} by {
    if k < n and target != Nat.0 and falling_product_target_primewise_bound(n, k, target) {
        let a: Nat = falling_product(n, k)
        falling_product_nonzero(n, k)
        a != Nat.0
        forall(p: Nat) {
            if p.is_prime {
                count_prime_factor_falling_product_le_target_of_target_primewise_bound(
                    p, n, k, target)
                count_prime_factor(p, a) <= count_prime_factor(p, target)
            }
        }
        count_prime_factor_le_imp_divides(a, target)
        a.divides(target)
        falling_product(n, k) = a
        falling_product(n, k).divides(target)
    }
}

/// Falling-product divisibility of a nonzero target implies the primewise
/// target valuation bounds.
theorem falling_product_target_primewise_bound_of_divides(
    n: Nat, k: Nat, target: Nat
) {
    k < n and target != Nat.0 and falling_product(n, k).divides(target)
        implies falling_product_target_primewise_bound(n, k, target)
} by {
    if k < n and target != Nat.0 and falling_product(n, k).divides(target) {
        falling_product_nonzero(n, k)
        forall(p: Nat) {
            if p.is_prime {
                divides_imp_count_prime_factor_le(p, falling_product(n, k), target)
                count_prime_factor(p, falling_product(n, k)) <= count_prime_factor(p, target)
                count_prime_factor_falling_product(p, n, k)
                count_prime_factor(p, falling_product(n, k)) =
                    falling_product_prime_count_sum(p, n, k)
                falling_product_prime_count_sum(p, n, k) <= count_prime_factor(p, target)
                falling_product_target_prime_bound(p, n, k, target)
            }
        }
        falling_product_target_primewise_bound(n, k, target)
    }
}

/// A positive falling product divides a nonzero target iff all prime valuation
/// sums are bounded by the matching target valuations.
theorem falling_product_divides_target_iff_target_primewise_bound(
    n: Nat, k: Nat, target: Nat
) {
    k < n and target != Nat.0 implies (
        falling_product(n, k).divides(target) =
        falling_product_target_primewise_bound(n, k, target)
    )
} by {
    if k < n and target != Nat.0 {
        let divides_target: Bool = falling_product(n, k).divides(target)
        let target_bound: Bool = falling_product_target_primewise_bound(n, k, target)
        if divides_target {
            falling_product_target_primewise_bound_of_divides(n, k, target)
            target_bound
        }
        if target_bound {
            falling_product_divides_target_of_target_primewise_bound(n, k, target)
            divides_target
        }
        divides_target = target_bound
        falling_product(n, k).divides(target) =
            falling_product_target_primewise_bound(n, k, target)
    }
}

/// A prime power divides a positive falling product iff its exponent is bounded
/// by the matching falling-product valuation sum.
theorem prime_pow_divides_falling_product_iff_prime_count_sum(
    p: Nat, exponent: Nat, n: Nat, k: Nat
) {
    p.is_prime and k < n implies (
        p.pow(exponent).divides(falling_product(n, k)) =
        (exponent <= falling_product_prime_count_sum(p, n, k))
    )
} by {
    if p.is_prime and k < n {
        falling_product_nonzero(n, k)
        prime_pow_divides_iff(p, exponent, falling_product(n, k))
        p.pow(exponent).divides(falling_product(n, k)) =
            (exponent <= count_prime_factor(p, falling_product(n, k)))
        count_prime_factor_falling_product(p, n, k)
        count_prime_factor(p, falling_product(n, k)) =
            falling_product_prime_count_sum(p, n, k)
        p.pow(exponent).divides(falling_product(n, k)) =
            (exponent <= falling_product_prime_count_sum(p, n, k))
    }
}

/// If a prime-power exponent is bounded by a falling-product valuation sum,
/// then that prime power divides the falling product.
theorem prime_pow_divides_falling_product_of_prime_count_sum_le(
    p: Nat, exponent: Nat, n: Nat, k: Nat
) {
    p.is_prime and k < n and exponent <= falling_product_prime_count_sum(p, n, k)
        implies p.pow(exponent).divides(falling_product(n, k))
} by {
    if p.is_prime and k < n and exponent <= falling_product_prime_count_sum(p, n, k) {
        prime_pow_divides_falling_product_iff_prime_count_sum(p, exponent, n, k)
        p.pow(exponent).divides(falling_product(n, k)) =
            (exponent <= falling_product_prime_count_sum(p, n, k))
        p.pow(exponent).divides(falling_product(n, k))
    }
}

/// If a prime power divides a positive falling product, then its exponent is
/// bounded by the matching falling-product valuation sum.
theorem prime_count_sum_le_of_prime_pow_divides_falling_product(
    p: Nat, exponent: Nat, n: Nat, k: Nat
) {
    p.is_prime and k < n and p.pow(exponent).divides(falling_product(n, k))
        implies exponent <= falling_product_prime_count_sum(p, n, k)
} by {
    if p.is_prime and k < n and p.pow(exponent).divides(falling_product(n, k)) {
        prime_pow_divides_falling_product_iff_prime_count_sum(p, exponent, n, k)
        p.pow(exponent).divides(falling_product(n, k)) =
            (exponent <= falling_product_prime_count_sum(p, n, k))
        exponent <= falling_product_prime_count_sum(p, n, k)
    }
}

/// The falling-product valuation sum is bounded by the central binomial
/// valuation at a prime.
define falling_product_central_binom_prime_bound(p: Nat, n: Nat, k: Nat) -> Bool {
    falling_product_prime_count_sum(p, n, k) <= count_prime_factor(p, central_binom(n))
}

/// The falling-product valuation sum is bounded by the central binomial carry
/// count at a prime base.
define falling_product_central_binom_carry_bound(p: Nat, n: Nat, k: Nat) -> Bool {
    falling_product_prime_count_sum(p, n, k) <= double_addition_carry_count(p, n)
}

/// Every prime valuation sum in the falling product is bounded by the matching
/// central binomial valuation.
define falling_product_central_binom_primewise_bound(n: Nat, k: Nat) -> Bool {
    forall(p: Nat) {
        p.is_prime implies falling_product_central_binom_prime_bound(p, n, k)
    }
}

/// Every prime valuation sum in the falling product is bounded by the matching
/// central binomial carry count.
define falling_product_central_binom_carrywise_bound(n: Nat, k: Nat) -> Bool {
    forall(p: Nat) {
        p.is_prime implies falling_product_central_binom_carry_bound(p, n, k)
    }
}

/// The binary falling-product valuation sum is bounded by the binary digit sum.
define falling_product_central_binom_binary_digit_bound(n: Nat, k: Nat) -> Bool {
    falling_product_prime_count_sum(Nat.2, n, k) <= digit_sum(Nat.2, n)
}

/// A prime power divides a central binomial coefficient iff its exponent is
/// bounded by the doubled-addend carry count.
theorem prime_pow_divides_central_binom_iff_double_addition_carry_count(
    p: Nat, exponent: Nat, n: Nat
) {
    p.is_prime implies (
        p.pow(exponent).divides(central_binom(n)) =
        (exponent <= double_addition_carry_count(p, n))
    )
} by {
    if p.is_prime {
        central_binom_ne_zero(n)
        prime_pow_divides_iff(p, exponent, central_binom(n))
        p.pow(exponent).divides(central_binom(n)) =
            (exponent <= count_prime_factor(p, central_binom(n)))
        central_binom_valuation_eq_double_addition_carry_count(p, n)
        count_prime_factor(p, central_binom(n)) =
            double_addition_carry_count(p, n)
        p.pow(exponent).divides(central_binom(n)) =
            (exponent <= double_addition_carry_count(p, n))
    }
}

/// A central-binomial carry-count lower bound implies divisibility by the
/// corresponding prime power.
theorem prime_pow_divides_central_binom_of_exponent_le_double_carry(
    p: Nat, exponent: Nat, n: Nat
) {
    p.is_prime and exponent <= double_addition_carry_count(p, n)
        implies p.pow(exponent).divides(central_binom(n))
} by {
    if p.is_prime and exponent <= double_addition_carry_count(p, n) {
        prime_pow_divides_central_binom_iff_double_addition_carry_count(
            p, exponent, n)
        p.pow(exponent).divides(central_binom(n)) =
            (exponent <= double_addition_carry_count(p, n))
        p.pow(exponent).divides(central_binom(n))
    }
}

/// Divisibility of a central binomial coefficient by a prime power bounds the
/// exponent by the doubled-addend carry count.
theorem exponent_le_double_carry_of_prime_pow_divides_central_binom(
    p: Nat, exponent: Nat, n: Nat
) {
    p.is_prime and p.pow(exponent).divides(central_binom(n))
        implies exponent <= double_addition_carry_count(p, n)
} by {
    if p.is_prime and p.pow(exponent).divides(central_binom(n)) {
        prime_pow_divides_central_binom_iff_double_addition_carry_count(
            p, exponent, n)
        p.pow(exponent).divides(central_binom(n)) =
            (exponent <= double_addition_carry_count(p, n))
        exponent <= double_addition_carry_count(p, n)
    }
}

/// A power of two divides a central binomial coefficient iff its exponent is
/// bounded by the binary digit sum.
theorem prime_pow_two_divides_central_binom_iff_digit_sum(
    exponent: Nat, n: Nat
) {
    Nat.2.pow(exponent).divides(central_binom(n)) =
        (exponent <= digit_sum(Nat.2, n))
} by {
    nat_two_prime
    prime_pow_divides_central_binom_iff_double_addition_carry_count(
        Nat.2, exponent, n)
    Nat.2.pow(exponent).divides(central_binom(n)) =
        (exponent <= double_addition_carry_count(Nat.2, n))
    double_addition_carry_count_two_eq_digit_sum(n)
    double_addition_carry_count(Nat.2, n) = digit_sum(Nat.2, n)
    Nat.2.pow(exponent).divides(central_binom(n)) =
        (exponent <= digit_sum(Nat.2, n))
}

/// A binary digit-sum lower bound implies divisibility of the central binomial
/// coefficient by the corresponding power of two.
theorem prime_pow_two_divides_central_binom_of_exponent_le_digit_sum(
    exponent: Nat, n: Nat
) {
    exponent <= digit_sum(Nat.2, n)
        implies Nat.2.pow(exponent).divides(central_binom(n))
} by {
    if exponent <= digit_sum(Nat.2, n) {
        prime_pow_two_divides_central_binom_iff_digit_sum(exponent, n)
        Nat.2.pow(exponent).divides(central_binom(n)) =
            (exponent <= digit_sum(Nat.2, n))
        Nat.2.pow(exponent).divides(central_binom(n))
    }
}

/// Divisibility of a central binomial coefficient by a power of two bounds the
/// exponent by the binary digit sum.
theorem exponent_le_digit_sum_of_prime_pow_two_divides_central_binom(
    exponent: Nat, n: Nat
) {
    Nat.2.pow(exponent).divides(central_binom(n))
        implies exponent <= digit_sum(Nat.2, n)
} by {
    if Nat.2.pow(exponent).divides(central_binom(n)) {
        prime_pow_two_divides_central_binom_iff_digit_sum(exponent, n)
        Nat.2.pow(exponent).divides(central_binom(n)) =
            (exponent <= digit_sum(Nat.2, n))
        exponent <= digit_sum(Nat.2, n)
    }
}

/// The central-binomial prime bound is the generic target bound specialized to
/// the central binomial coefficient.
theorem falling_product_central_binom_prime_bound_eq_target_prime_bound(
    p: Nat, n: Nat, k: Nat
) {
    falling_product_central_binom_prime_bound(p, n, k) =
        falling_product_target_prime_bound(p, n, k, central_binom(n))
}

/// The generic target bound specialized to the central binomial coefficient is
/// the central-binomial prime bound.
theorem falling_product_target_prime_bound_eq_central_binom_prime_bound(
    p: Nat, n: Nat, k: Nat
) {
    falling_product_target_prime_bound(p, n, k, central_binom(n)) =
        falling_product_central_binom_prime_bound(p, n, k)
}

/// Central-binomial primewise bounds imply the generic target bounds
/// specialized to the central binomial coefficient.
theorem falling_product_target_primewise_bound_of_central_binom_primewise_bound(
    n: Nat, k: Nat
) {
    falling_product_central_binom_primewise_bound(n, k)
        implies falling_product_target_primewise_bound(n, k, central_binom(n))
} by {
    if falling_product_central_binom_primewise_bound(n, k) {
        forall(p: Nat) {
            if p.is_prime {
                let h: Bool = p.is_prime implies falling_product_central_binom_prime_bound(p, n, k)
                h
                falling_product_central_binom_prime_bound(p, n, k)
                falling_product_target_prime_bound_eq_central_binom_prime_bound(p, n, k)
                falling_product_target_prime_bound(p, n, k, central_binom(n))
            }
        }
        falling_product_target_primewise_bound(n, k, central_binom(n))
    }
}

/// Generic target bounds specialized to the central binomial coefficient imply
/// central-binomial primewise bounds.
theorem falling_product_central_binom_primewise_bound_of_target_primewise_bound(
    n: Nat, k: Nat
) {
    falling_product_target_primewise_bound(n, k, central_binom(n))
        implies falling_product_central_binom_primewise_bound(n, k)
} by {
    if falling_product_target_primewise_bound(n, k, central_binom(n)) {
        forall(p: Nat) {
            if p.is_prime {
                falling_product_target_prime_bound_of_primewise_bound(
                    p, n, k, central_binom(n))
                falling_product_target_prime_bound(p, n, k, central_binom(n))
                falling_product_central_binom_prime_bound_eq_target_prime_bound(p, n, k)
                falling_product_central_binom_prime_bound(p, n, k)
            }
        }
        falling_product_central_binom_primewise_bound(n, k)
    }
}

/// Central-binomial primewise bounds are equivalent to generic target bounds
/// specialized to the central binomial coefficient.
theorem falling_product_central_binom_primewise_bound_iff_target_primewise_bound(
    n: Nat, k: Nat
) {
    falling_product_central_binom_primewise_bound(n, k) =
        falling_product_target_primewise_bound(n, k, central_binom(n))
} by {
    if falling_product_central_binom_primewise_bound(n, k) {
        falling_product_target_primewise_bound_of_central_binom_primewise_bound(n, k)
        falling_product_target_primewise_bound(n, k, central_binom(n))
    }
    if falling_product_target_primewise_bound(n, k, central_binom(n)) {
        falling_product_central_binom_primewise_bound_of_target_primewise_bound(n, k)
        falling_product_central_binom_primewise_bound(n, k)
    }
}

/// Generic target bounds specialized to the central binomial coefficient imply
/// divisibility of the central binomial coefficient by the falling product.
theorem falling_product_divides_central_binom_of_target_primewise_bound(
    n: Nat, k: Nat
) {
    k < n and falling_product_target_primewise_bound(n, k, central_binom(n))
        implies falling_product(n, k).divides(central_binom(n))
} by {
    if k < n and falling_product_target_primewise_bound(n, k, central_binom(n)) {
        central_binom_ne_zero(n)
        falling_product_divides_target_of_target_primewise_bound(
            n, k, central_binom(n))
        falling_product(n, k).divides(central_binom(n))
    }
}

/// Falling-product divisibility of the central binomial coefficient implies
/// the generic target bounds specialized to the central binomial coefficient.
theorem falling_product_target_primewise_bound_of_divides_central_binom(
    n: Nat, k: Nat
) {
    k < n and falling_product(n, k).divides(central_binom(n))
        implies falling_product_target_primewise_bound(n, k, central_binom(n))
} by {
    if k < n and falling_product(n, k).divides(central_binom(n)) {
        central_binom_ne_zero(n)
        falling_product_target_primewise_bound_of_divides(
            n, k, central_binom(n))
        falling_product_target_primewise_bound(n, k, central_binom(n))
    }
}

/// A positive falling product divides the central binomial coefficient iff the
/// generic target bounds hold for the central binomial coefficient.
theorem falling_product_divides_central_binom_iff_target_primewise_bound(
    n: Nat, k: Nat
) {
    k < n implies (
        falling_product(n, k).divides(central_binom(n)) =
        falling_product_target_primewise_bound(n, k, central_binom(n))
    )
} by {
    if k < n {
        let divides_central: Bool = falling_product(n, k).divides(central_binom(n))
        let target_bound: Bool = falling_product_target_primewise_bound(n, k, central_binom(n))
        if divides_central {
            falling_product_target_primewise_bound_of_divides_central_binom(n, k)
            target_bound
        }
        if target_bound {
            falling_product_divides_central_binom_of_target_primewise_bound(n, k)
            divides_central
        }
        divides_central = target_bound
        falling_product(n, k).divides(central_binom(n)) =
            falling_product_target_primewise_bound(n, k, central_binom(n))
    }
}

/// The central binomial valuation bound implies the corresponding carry-count
/// bound.
theorem falling_product_central_binom_carry_bound_of_prime_bound(
    p: Nat, n: Nat, k: Nat
) {
    p.is_prime and falling_product_central_binom_prime_bound(p, n, k)
        implies falling_product_central_binom_carry_bound(p, n, k)
} by {
    if p.is_prime and falling_product_central_binom_prime_bound(p, n, k) {
        central_binom_valuation_eq_double_addition_carry_count(p, n)
        count_prime_factor(p, central_binom(n)) =
            double_addition_carry_count(p, n)
        falling_product_prime_count_sum(p, n, k) <= count_prime_factor(p, central_binom(n))
        falling_product_prime_count_sum(p, n, k) <= double_addition_carry_count(p, n)
        falling_product_central_binom_carry_bound(p, n, k)
    }
}

/// The central binomial carry-count bound implies the corresponding valuation
/// bound.
theorem falling_product_central_binom_prime_bound_of_carry_bound(
    p: Nat, n: Nat, k: Nat
) {
    p.is_prime and falling_product_central_binom_carry_bound(p, n, k)
        implies falling_product_central_binom_prime_bound(p, n, k)
} by {
    if p.is_prime and falling_product_central_binom_carry_bound(p, n, k) {
        double_addition_carry_count_eq_central_binom_valuation(p, n)
        double_addition_carry_count(p, n) =
            count_prime_factor(p, central_binom(n))
        falling_product_prime_count_sum(p, n, k) <= double_addition_carry_count(p, n)
        falling_product_prime_count_sum(p, n, k) <= count_prime_factor(p, central_binom(n))
        falling_product_central_binom_prime_bound(p, n, k)
    }
}

/// The central binomial valuation bound and carry-count bound are equivalent
/// at a prime.
theorem falling_product_central_binom_prime_bound_iff_carry_bound(
    p: Nat, n: Nat, k: Nat
) {
    p.is_prime implies (
        falling_product_central_binom_prime_bound(p, n, k) =
        falling_product_central_binom_carry_bound(p, n, k)
    )
} by {
    if p.is_prime {
        if falling_product_central_binom_prime_bound(p, n, k) {
            falling_product_central_binom_carry_bound_of_prime_bound(p, n, k)
        }
        if falling_product_central_binom_carry_bound(p, n, k) {
            falling_product_central_binom_prime_bound_of_carry_bound(p, n, k)
        }
    }
}

/// A primewise central binomial valuation bound gives the valuation bound at
/// any particular prime.
theorem falling_product_central_binom_prime_bound_of_primewise_bound(
    p: Nat, n: Nat, k: Nat
) {
    p.is_prime and falling_product_central_binom_primewise_bound(n, k)
        implies falling_product_central_binom_prime_bound(p, n, k)
} by {
    if p.is_prime and falling_product_central_binom_primewise_bound(n, k) {
        let h: Bool = p.is_prime implies falling_product_central_binom_prime_bound(p, n, k)
        h
        falling_product_central_binom_prime_bound(p, n, k)
    }
}

/// A primewise central binomial carry-count bound gives the carry-count bound
/// at any particular prime.
theorem falling_product_central_binom_carry_bound_of_carrywise_bound(
    p: Nat, n: Nat, k: Nat
) {
    p.is_prime and falling_product_central_binom_carrywise_bound(n, k)
        implies falling_product_central_binom_carry_bound(p, n, k)
} by {
    if p.is_prime and falling_product_central_binom_carrywise_bound(n, k) {
        let h: Bool = p.is_prime implies falling_product_central_binom_carry_bound(p, n, k)
        h
        falling_product_central_binom_carry_bound(p, n, k)
    }
}

/// Primewise central binomial valuation bounds imply primewise carry-count
/// bounds.
theorem falling_product_central_binom_carrywise_bound_of_primewise_bound(
    n: Nat, k: Nat
) {
    falling_product_central_binom_primewise_bound(n, k)
        implies falling_product_central_binom_carrywise_bound(n, k)
} by {
    if falling_product_central_binom_primewise_bound(n, k) {
        forall(p: Nat) {
            if p.is_prime {
                falling_product_central_binom_prime_bound_of_primewise_bound(p, n, k)
                falling_product_central_binom_carry_bound_of_prime_bound(p, n, k)
                falling_product_central_binom_carry_bound(p, n, k)
            }
        }
        falling_product_central_binom_carrywise_bound(n, k)
    }
}

/// Primewise central binomial carry-count bounds imply primewise valuation
/// bounds.
theorem falling_product_central_binom_primewise_bound_of_carrywise_bound(
    n: Nat, k: Nat
) {
    falling_product_central_binom_carrywise_bound(n, k)
        implies falling_product_central_binom_primewise_bound(n, k)
} by {
    if falling_product_central_binom_carrywise_bound(n, k) {
        forall(p: Nat) {
            if p.is_prime {
                falling_product_central_binom_carry_bound_of_carrywise_bound(p, n, k)
                falling_product_central_binom_prime_bound_of_carry_bound(p, n, k)
                falling_product_central_binom_prime_bound(p, n, k)
            }
        }
        falling_product_central_binom_primewise_bound(n, k)
    }
}

/// The primewise central binomial valuation bounds are equivalent to the
/// primewise carry-count bounds.
theorem falling_product_central_binom_primewise_bound_iff_carrywise_bound(
    n: Nat, k: Nat
) {
    falling_product_central_binom_primewise_bound(n, k) =
        falling_product_central_binom_carrywise_bound(n, k)
} by {
    if falling_product_central_binom_primewise_bound(n, k) {
        falling_product_central_binom_carrywise_bound_of_primewise_bound(n, k)
    }
    if falling_product_central_binom_carrywise_bound(n, k) {
        falling_product_central_binom_primewise_bound_of_carrywise_bound(n, k)
    }
}

/// Generic target bounds for the central binomial coefficient are equivalent
/// to central-binomial carrywise bounds.
theorem falling_product_target_primewise_bound_iff_carrywise_bound(
    n: Nat, k: Nat
) {
    falling_product_target_primewise_bound(n, k, central_binom(n)) =
        falling_product_central_binom_carrywise_bound(n, k)
} by {
    if falling_product_target_primewise_bound(n, k, central_binom(n)) {
        falling_product_central_binom_primewise_bound_of_target_primewise_bound(n, k)
        falling_product_central_binom_primewise_bound(n, k)
        falling_product_central_binom_carrywise_bound_of_primewise_bound(n, k)
        falling_product_central_binom_carrywise_bound(n, k)
    }
    if falling_product_central_binom_carrywise_bound(n, k) {
        falling_product_central_binom_primewise_bound_of_carrywise_bound(n, k)
        falling_product_central_binom_primewise_bound(n, k)
        falling_product_target_primewise_bound_of_central_binom_primewise_bound(n, k)
        falling_product_target_primewise_bound(n, k, central_binom(n))
    }
}

/// A central binomial valuation bound for one prime bounds the matching
/// valuation of the falling product itself.
theorem count_prime_factor_falling_product_le_central_binom_of_prime_bound(
    p: Nat, n: Nat, k: Nat
) {
    k < n and falling_product_central_binom_prime_bound(p, n, k)
        implies count_prime_factor(p, falling_product(n, k)) <= count_prime_factor(p, central_binom(n))
} by {
    if k < n and falling_product_central_binom_prime_bound(p, n, k) {
        falling_product_prime_count_sum(p, n, k) <= count_prime_factor(p, central_binom(n))
        count_prime_factor_falling_product(p, n, k)
        count_prime_factor(p, falling_product(n, k)) =
            falling_product_prime_count_sum(p, n, k)
        count_prime_factor(p, falling_product(n, k)) <= count_prime_factor(p, central_binom(n))
    }
}

/// Primewise central binomial valuation bounds give the prime-count
/// inequalities needed for divisibility.
theorem count_prime_factor_falling_product_le_central_binom_of_primewise_bound(
    p: Nat, n: Nat, k: Nat
) {
    p.is_prime and k < n and falling_product_central_binom_primewise_bound(n, k)
        implies count_prime_factor(p, falling_product(n, k)) <= count_prime_factor(p, central_binom(n))
} by {
    if p.is_prime and k < n and falling_product_central_binom_primewise_bound(n, k) {
        falling_product_central_binom_prime_bound_of_primewise_bound(p, n, k)
        falling_product_central_binom_prime_bound(p, n, k)
        count_prime_factor_falling_product_le_central_binom_of_prime_bound(p, n, k)
        count_prime_factor(p, falling_product(n, k)) <= count_prime_factor(p, central_binom(n))
    }
}

/// Primewise central binomial valuation bounds imply divisibility of the
/// central binomial coefficient by the falling product.
theorem falling_product_divides_central_binom_of_primewise_bound(n: Nat, k: Nat) {
    k < n and falling_product_central_binom_primewise_bound(n, k)
        implies falling_product(n, k).divides(central_binom(n))
} by {
    if k < n and falling_product_central_binom_primewise_bound(n, k) {
        let a: Nat = falling_product(n, k)
        let b: Nat = central_binom(n)
        falling_product_nonzero(n, k)
        a != Nat.0
        central_binom_ne_zero(n)
        b != Nat.0
        forall(p: Nat) {
            if p.is_prime {
                count_prime_factor_falling_product_le_central_binom_of_primewise_bound(p, n, k)
                count_prime_factor(p, a) <= count_prime_factor(p, b)
            }
        }
        count_prime_factor_le_imp_divides(a, b)
        a.divides(b)
        falling_product(n, k) = a
        central_binom(n) = b
        falling_product(n, k).divides(central_binom(n))
    }
}

/// Primewise central binomial carry-count bounds imply divisibility of the
/// central binomial coefficient by the falling product.
theorem falling_product_divides_central_binom_of_carrywise_bound(n: Nat, k: Nat) {
    k < n and falling_product_central_binom_carrywise_bound(n, k)
        implies falling_product(n, k).divides(central_binom(n))
} by {
    if k < n and falling_product_central_binom_carrywise_bound(n, k) {
        falling_product_central_binom_primewise_bound_of_carrywise_bound(n, k)
        falling_product_central_binom_primewise_bound(n, k)
        falling_product_divides_central_binom_of_primewise_bound(n, k)
        falling_product(n, k).divides(central_binom(n))
    }
}

/// Falling-product divisibility of the central binomial coefficient implies
/// the primewise valuation bounds.
theorem falling_product_central_binom_primewise_bound_of_divides(
    n: Nat, k: Nat
) {
    k < n and falling_product(n, k).divides(central_binom(n))
        implies falling_product_central_binom_primewise_bound(n, k)
} by {
    if k < n and falling_product(n, k).divides(central_binom(n)) {
        falling_product_nonzero(n, k)
        central_binom_ne_zero(n)
        forall(p: Nat) {
            if p.is_prime {
                divides_imp_count_prime_factor_le(p, falling_product(n, k), central_binom(n))
                count_prime_factor(p, falling_product(n, k)) <= count_prime_factor(p, central_binom(n))
                count_prime_factor_falling_product(p, n, k)
                count_prime_factor(p, falling_product(n, k)) =
                    falling_product_prime_count_sum(p, n, k)
                falling_product_prime_count_sum(p, n, k) <= count_prime_factor(p, central_binom(n))
                falling_product_central_binom_prime_bound(p, n, k)
            }
        }
        falling_product_central_binom_primewise_bound(n, k)
    }
}

/// Falling-product divisibility of the central binomial coefficient implies
/// the primewise carry-count bounds.
theorem falling_product_central_binom_carrywise_bound_of_divides(
    n: Nat, k: Nat
) {
    k < n and falling_product(n, k).divides(central_binom(n))
        implies falling_product_central_binom_carrywise_bound(n, k)
} by {
    if k < n and falling_product(n, k).divides(central_binom(n)) {
        falling_product_central_binom_primewise_bound_of_divides(n, k)
        falling_product_central_binom_primewise_bound(n, k)
        falling_product_central_binom_carrywise_bound_of_primewise_bound(n, k)
        falling_product_central_binom_carrywise_bound(n, k)
    }
}

/// A positive falling product divides the central binomial coefficient iff all
/// prime valuation sums are bounded by the matching central binomial
/// valuations.
theorem falling_product_divides_central_binom_iff_primewise_bound(n: Nat, k: Nat) {
    k < n implies (
        falling_product(n, k).divides(central_binom(n)) =
        falling_product_central_binom_primewise_bound(n, k)
    )
} by {
    if k < n {
        if falling_product(n, k).divides(central_binom(n)) {
            falling_product_central_binom_primewise_bound_of_divides(n, k)
        }
        if falling_product_central_binom_primewise_bound(n, k) {
            falling_product_divides_central_binom_of_primewise_bound(n, k)
        }
    }
}

/// A positive falling product divides the central binomial coefficient iff all
/// prime valuation sums are bounded by the matching central binomial carry
/// counts.
theorem falling_product_divides_central_binom_iff_carrywise_bound(n: Nat, k: Nat) {
    k < n implies (
        falling_product(n, k).divides(central_binom(n)) =
        falling_product_central_binom_carrywise_bound(n, k)
    )
} by {
    if k < n {
        if falling_product(n, k).divides(central_binom(n)) {
            falling_product_central_binom_carrywise_bound_of_divides(n, k)
        }
        if falling_product_central_binom_carrywise_bound(n, k) {
            falling_product_divides_central_binom_of_carrywise_bound(n, k)
        }
    }
}

/// The binary digit-sum bound implies the binary carry-count bound.
theorem falling_product_central_binom_carry_bound_two_of_binary_digit_bound(
    n: Nat, k: Nat
) {
    falling_product_central_binom_binary_digit_bound(n, k)
        implies falling_product_central_binom_carry_bound(Nat.2, n, k)
} by {
    if falling_product_central_binom_binary_digit_bound(n, k) {
        digit_sum_eq_double_addition_carry_count_two(n)
        digit_sum(Nat.2, n) = double_addition_carry_count(Nat.2, n)
        falling_product_prime_count_sum(Nat.2, n, k) <= digit_sum(Nat.2, n)
        falling_product_prime_count_sum(Nat.2, n, k) <= double_addition_carry_count(Nat.2, n)
        falling_product_central_binom_carry_bound(Nat.2, n, k)
    }
}

/// The binary carry-count bound implies the binary digit-sum bound.
theorem falling_product_central_binom_binary_digit_bound_of_carry_bound_two(
    n: Nat, k: Nat
) {
    falling_product_central_binom_carry_bound(Nat.2, n, k)
        implies falling_product_central_binom_binary_digit_bound(n, k)
} by {
    if falling_product_central_binom_carry_bound(Nat.2, n, k) {
        double_addition_carry_count_two_eq_digit_sum(n)
        double_addition_carry_count(Nat.2, n) = digit_sum(Nat.2, n)
        falling_product_prime_count_sum(Nat.2, n, k) <= double_addition_carry_count(Nat.2, n)
        falling_product_prime_count_sum(Nat.2, n, k) <= digit_sum(Nat.2, n)
        falling_product_central_binom_binary_digit_bound(n, k)
    }
}

/// The binary digit-sum bound and binary carry-count bound are equivalent.
theorem falling_product_central_binom_binary_digit_bound_iff_carry_bound_two(
    n: Nat, k: Nat
) {
    falling_product_central_binom_binary_digit_bound(n, k) =
        falling_product_central_binom_carry_bound(Nat.2, n, k)
} by {
    if falling_product_central_binom_binary_digit_bound(n, k) {
        falling_product_central_binom_carry_bound_two_of_binary_digit_bound(n, k)
    }
    if falling_product_central_binom_carry_bound(Nat.2, n, k) {
        falling_product_central_binom_binary_digit_bound_of_carry_bound_two(n, k)
    }
}

/// The binary digit-sum bound is equivalent to the binary central binomial
/// valuation bound.
theorem falling_product_central_binom_binary_digit_bound_iff_prime_bound_two(
    n: Nat, k: Nat
) {
    falling_product_central_binom_binary_digit_bound(n, k) =
        falling_product_central_binom_prime_bound(Nat.2, n, k)
} by {
    nat_two_prime
    if falling_product_central_binom_binary_digit_bound(n, k) {
        central_binom_two_adic_valuation(n)
        count_prime_factor(Nat.2, central_binom(n)) = digit_sum(Nat.2, n)
        falling_product_prime_count_sum(Nat.2, n, k) <= digit_sum(Nat.2, n)
        falling_product_prime_count_sum(Nat.2, n, k) <= count_prime_factor(Nat.2, central_binom(n))
        falling_product_central_binom_prime_bound(Nat.2, n, k)
    }
    if falling_product_central_binom_prime_bound(Nat.2, n, k) {
        central_binom_two_adic_valuation(n)
        count_prime_factor(Nat.2, central_binom(n)) = digit_sum(Nat.2, n)
        falling_product_prime_count_sum(Nat.2, n, k) <= count_prime_factor(Nat.2, central_binom(n))
        falling_product_prime_count_sum(Nat.2, n, k) <= digit_sum(Nat.2, n)
        falling_product_central_binom_binary_digit_bound(n, k)
    }
}

/// The central binomial carry bound for the top falling-product factor.
theorem falling_product_central_binom_carry_bound_zero(p: Nat, n: Nat) {
    falling_product_central_binom_carry_bound(p, n, Nat.0) =
        (count_prime_factor(p, n) <= double_addition_carry_count(p, n))
} by {
    falling_product_prime_count_sum_zero(p, n)
}

/// The central binomial prime-valuation bound for the top falling-product
/// factor.
theorem falling_product_central_binom_prime_bound_zero(p: Nat, n: Nat) {
    falling_product_central_binom_prime_bound(p, n, Nat.0) =
        (count_prime_factor(p, n) <= count_prime_factor(p, central_binom(n)))
} by {
    falling_product_prime_count_sum_zero(p, n)
}

/// The binary digit-sum bound for the top falling-product factor.
theorem falling_product_central_binom_binary_digit_bound_zero(n: Nat) {
    falling_product_central_binom_binary_digit_bound(n, Nat.0) =
        (count_prime_factor(Nat.2, n) <= digit_sum(Nat.2, n))
} by {
    falling_product_prime_count_sum_zero(Nat.2, n)
}

/// The central binomial carry bound for the two-factor falling product.
theorem falling_product_central_binom_carry_bound_one(p: Nat, n: Nat) {
    falling_product_central_binom_carry_bound(p, n, Nat.1) =
        (count_prime_factor(p, n) + count_prime_factor(p, n - Nat.1) <= double_addition_carry_count(p, n))
} by {
    falling_product_prime_count_sum_one(p, n)
}

/// The central binomial prime-valuation bound for the two-factor falling
/// product.
theorem falling_product_central_binom_prime_bound_one(p: Nat, n: Nat) {
    falling_product_central_binom_prime_bound(p, n, Nat.1) =
        (count_prime_factor(p, n) + count_prime_factor(p, n - Nat.1) <= count_prime_factor(p, central_binom(n)))
} by {
    falling_product_prime_count_sum_one(p, n)
}

/// The binary digit-sum bound for the two-factor falling product.
theorem falling_product_central_binom_binary_digit_bound_one(n: Nat) {
    falling_product_central_binom_binary_digit_bound(n, Nat.1) =
        (count_prime_factor(Nat.2, n) + count_prime_factor(Nat.2, n - Nat.1) <= digit_sum(Nat.2, n))
} by {
    falling_product_prime_count_sum_one(Nat.2, n)
}

/// The central binomial carry bound for the three-factor falling product.
theorem falling_product_central_binom_carry_bound_two(p: Nat, n: Nat) {
    falling_product_central_binom_carry_bound(p, n, Nat.2) =
        (count_prime_factor(p, n) + count_prime_factor(p, n - Nat.1) +
            count_prime_factor(p, n - Nat.2) <= double_addition_carry_count(p, n))
} by {
    falling_product_prime_count_sum_two(p, n)
}

/// The central binomial prime-valuation bound for the three-factor falling
/// product.
theorem falling_product_central_binom_prime_bound_two(p: Nat, n: Nat) {
    falling_product_central_binom_prime_bound(p, n, Nat.2) =
        (count_prime_factor(p, n) + count_prime_factor(p, n - Nat.1) +
            count_prime_factor(p, n - Nat.2) <= count_prime_factor(p, central_binom(n)))
} by {
    falling_product_prime_count_sum_two(p, n)
}

/// The binary digit-sum bound for the three-factor falling product.
theorem falling_product_central_binom_binary_digit_bound_two(n: Nat) {
    falling_product_central_binom_binary_digit_bound(n, Nat.2) =
        (count_prime_factor(Nat.2, n) + count_prime_factor(Nat.2, n - Nat.1) +
            count_prime_factor(Nat.2, n - Nat.2) <= digit_sum(Nat.2, n))
} by {
    falling_product_prime_count_sum_two(Nat.2, n)
}
