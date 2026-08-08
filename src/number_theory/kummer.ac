from nat import Nat, digit_sum, add_cancels_right, add_assoc, add_comm, add_comm_4,
    div_mod_decomp, digit_sum_recurrence, digit_sum_zero, div_lt, strong_induction,
    true_below, true_below_apply, factorial_zero
from number_theory.factorisation import count_prime_factor, count_prime_factor_one
from number_theory.legendre import legendre_binom_factorial
from number_theory.legendre_recurrence import legendre_factorial_recurrence
from combinatorics import binom
numerals Nat

/// The digit-sum form of Legendre's formula at a single natural number.
define legendre_digit_sum_at(p: Nat, n: Nat) -> Bool {
    p * count_prime_factor(p, n.factorial) + digit_sum(p, n) =
        n + count_prime_factor(p, n.factorial)
}

/// The algebraic step used to combine the digit-sum and factorial recurrences.
theorem legendre_digit_sum_step_algebra(p: Nat, q: Nat, r: Nat, v: Nat, s: Nat) {
    p * v + s = q + v implies
        p * (q + v) + (r + s) = q * p + r + (q + v)
} by {
    if p * v + s = q + v {
        p * (q + v) = p * q + p * v
        p * q = q * p
        p * (q + v) + (r + s) = q * p + r + (p * v + s)
        p * (q + v) + (r + s) = q * p + r + (q + v)
    }
}

/// The digit-sum form of Legendre's formula.
theorem legendre_digit_sum(p: Nat, n: Nat) {
    p.is_prime implies legendre_digit_sum_at(p, n)
} by {
    if p.is_prime {
        Nat.1 < p
        let f: Nat -> Bool = function(m: Nat) {
            legendre_digit_sum_at(p, m)
        }
        strong_induction(f)
        forall(m: Nat) {
            if true_below(f, m) {
                if m = Nat.0 {
                    factorial_zero
                    digit_sum_zero(p)
                    count_prime_factor_one(p)
                    m.factorial = Nat.1
                    count_prime_factor(p, m.factorial) = Nat.0
                    digit_sum(p, m) = Nat.0
                    p * count_prime_factor(p, m.factorial) = Nat.0
                    p * count_prime_factor(p, m.factorial) + digit_sum(p, m) = Nat.0
                    m + count_prime_factor(p, m.factorial) = Nat.0
                    p * count_prime_factor(p, m.factorial) + digit_sum(p, m) =
                        m + count_prime_factor(p, m.factorial)
                    legendre_digit_sum_at(p, m)
                }
                if m != Nat.0 {
                    let q: Nat = m.div(p)
                    let r: Nat = m.mod(p)
                    let v: Nat = count_prime_factor(p, q.factorial)
                    let s: Nat = digit_sum(p, q)
                    div_lt(m, p)
                    true_below_apply(f, m, q)
                    f(q)
                    legendre_digit_sum_at(p, q)
                    legendre_digit_sum_at(p, q) = (p * v + s = q + v)
                    p * v + s = q + v
                    legendre_factorial_recurrence(p, m)
                    count_prime_factor(p, m.factorial) = q + v
                    digit_sum_recurrence(p, m)
                    digit_sum(p, m) = r + s
                    div_mod_decomp(m, p)
                    q * p + r = m
                    legendre_digit_sum_step_algebra(p, q, r, v, s)
                    p * (q + v) + (r + s) = q * p + r + (q + v)
                    p * count_prime_factor(p, m.factorial) + digit_sum(p, m) =
                        m + count_prime_factor(p, m.factorial)
                    legendre_digit_sum_at(p, m)
                }
                f(m)
            }
        }
        f(n)
        legendre_digit_sum_at(p, n)
    }
}

/// A four-term addition shuffle used by Kummer's algebraic reduction.
theorem kummer_add_shuffle_left(x: Nat, y: Nat, z: Nat, w: Nat) {
    (x + y) + (z + w) = x + z + w + y
} by {
    add_comm_4(x, y, z, w)
    (x + y) + (z + w) = (x + z) + (y + w)
    add_comm(y, w)
    y + w = w + y
    (x + z) + (y + w) = (x + z) + (w + y)
    add_assoc(x + z, w, y)
    (x + z) + w + y = (x + z) + (w + y)
    (x + y) + (z + w) = x + z + w + y
}

/// A five-term addition shuffle used by Kummer's algebraic reduction.
theorem kummer_add_shuffle_right(vc: Nat, a: Nat, va: Nat, b: Nat, vb: Nat) {
    vc + (a + va) + (b + vb) = a + b + (vc + va + vb)
} by {
    add_assoc(vc, a, va)
    vc + (a + va) = vc + a + va
    add_comm(vc, a)
    vc + a = a + vc
    vc + a + va = a + vc + va
    add_assoc(a, vc, va)
    a + vc + va = a + (vc + va)
    vc + (a + va) = a + (vc + va)
    vc + (a + va) + (b + vb) = a + (vc + va) + (b + vb)
    add_comm_4(a, vc + va, b, vb)
    (a + (vc + va)) + (b + vb) = (a + b) + ((vc + va) + vb)
    (vc + va) + vb = vc + va + vb
    (a + b) + ((vc + va) + vb) = a + b + (vc + va + vb)
    vc + (a + va) + (b + vb) = a + b + (vc + va + vb)
}

/// Auxiliary algebra for adding the factorial valuation terms to Kummer's left side.
theorem kummer_left_carry_algebra(
    p: Nat, vc: Nat, va: Nat, vb: Nat, vab: Nat, sab: Nat
) {
    vc + va + vb = vab implies
        (p * vc + sab) + (p * va + p * vb) = p * vab + sab
} by {
    if vc + va + vb = vab {
        p * vc + p * va + p * vb = p * (vc + va + vb)
        p * vc + p * va + p * vb = p * vab
        kummer_add_shuffle_left(p * vc, sab, p * va, p * vb)
        (p * vc + sab) + (p * va + p * vb) =
            p * vc + p * va + p * vb + sab
        (p * vc + sab) + (p * va + p * vb) = p * vab + sab
    }
}

/// Auxiliary algebra for adding the factorial valuation terms to Kummer's right side.
theorem kummer_right_carry_algebra(
    p: Nat, a: Nat, b: Nat, vc: Nat, va: Nat, vb: Nat, vab: Nat, sa: Nat, sb: Nat
) {
    vc + va + vb = vab and
    p * va + sa = a + va and
    p * vb + sb = b + vb implies
        a + b + vab = (vc + sa + sb) + (p * va + p * vb)
} by {
    if vc + va + vb = vab and
        p * va + sa = a + va and
        p * vb + sb = b + vb {
        sa + p * va = a + va
        sb + p * vb = b + vb
        (vc + sa + sb) + (p * va + p * vb) =
            vc + (sa + p * va) + (sb + p * vb)
        (vc + sa + sb) + (p * va + p * vb) =
            vc + (a + va) + (b + vb)
        kummer_add_shuffle_right(vc, a, va, b, vb)
        vc + (a + va) + (b + vb) = a + b + (vc + va + vb)
        (vc + sa + sb) + (p * va + p * vb) = a + b + (vc + va + vb)
        (vc + sa + sb) + (p * va + p * vb) = a + b + vab
        a + b + vab = (vc + sa + sb) + (p * va + p * vb)
    }
}

/// The digit-sum form of Legendre's formula at `a`, `b`, and `a+b`
/// implies Kummer's digit-sum identity for `binom(a+b,a)`.
theorem kummer_digit_sum_of_legendre_digit_sum(
    p: Nat, a: Nat, b: Nat
) {
    legendre_digit_sum_at(p, a) and
    legendre_digit_sum_at(p, b) and
    legendre_digit_sum_at(p, a + b)
        implies
    p * count_prime_factor(p, (a + b).binom(a)) + digit_sum(p, a + b) =
        count_prime_factor(p, (a + b).binom(a)) + digit_sum(p, a) + digit_sum(p, b)
} by {
    if legendre_digit_sum_at(p, a) and
        legendre_digit_sum_at(p, b) and
        legendre_digit_sum_at(p, a + b) {
        let va: Nat = count_prime_factor(p, a.factorial)
        let vb: Nat = count_prime_factor(p, b.factorial)
        let vab: Nat = count_prime_factor(p, (a + b).factorial)
        let vc: Nat = count_prime_factor(p, (a + b).binom(a))
        let sa: Nat = digit_sum(p, a)
        let sb: Nat = digit_sum(p, b)
        let sab: Nat = digit_sum(p, a + b)
        let carry_terms: Nat = p * va + p * vb

        legendre_binom_factorial(p, a, b)
        vc + va + vb = vab
        legendre_digit_sum_at(p, a) =
            (p * va + sa = a + va)
        legendre_digit_sum_at(p, b) =
            (p * vb + sb = b + vb)
        legendre_digit_sum_at(p, a + b) =
            (p * vab + sab = a + b + vab)
        p * va + sa = a + va
        p * vb + sb = b + vb
        p * vab + sab = a + b + vab

        kummer_left_carry_algebra(p, vc, va, vb, vab, sab)
        (p * vc + sab) + carry_terms = p * vab + sab
        p * vab + sab = a + b + vab
        kummer_right_carry_algebra(p, a, b, vc, va, vb, vab, sa, sb)
        a + b + vab = (vc + sa + sb) + carry_terms
        (p * vc + sab) + carry_terms = (vc + sa + sb) + carry_terms
        add_cancels_right(carry_terms, p * vc + sab, vc + sa + sb)
        p * vc + sab = vc + sa + sb
        p * count_prime_factor(p, (a + b).binom(a)) + digit_sum(p, a + b) =
            p * vc + sab
        count_prime_factor(p, (a + b).binom(a)) + digit_sum(p, a) + digit_sum(p, b) =
            vc + sa + sb
        p * count_prime_factor(p, (a + b).binom(a)) + digit_sum(p, a + b) =
            count_prime_factor(p, (a + b).binom(a)) + digit_sum(p, a) + digit_sum(p, b)
    }
}

/// Kummer's theorem in additive digit-sum form. For a prime `p`, the p-adic
/// valuation of `binom(a+b,a)` satisfies the digit-sum carry equation
/// `p * v_p(binom(a+b,a)) + s_p(a+b) = v_p(binom(a+b,a)) + s_p(a) + s_p(b)`,
/// equivalently `(p-1) * v_p(binom(a+b,a)) = s_p(a) + s_p(b) - s_p(a+b)`, so the
/// valuation equals the number of carries when adding `a` and `b` in base `p`.
theorem kummer_digit_sum(p: Nat, a: Nat, b: Nat) {
    p.is_prime implies
        p * count_prime_factor(p, (a + b).binom(a)) + digit_sum(p, a + b) =
            count_prime_factor(p, (a + b).binom(a)) + digit_sum(p, a) + digit_sum(p, b)
} by {
    if p.is_prime {
        legendre_digit_sum(p, a)
        legendre_digit_sum(p, b)
        legendre_digit_sum(p, a + b)
        legendre_digit_sum_at(p, a)
        legendre_digit_sum_at(p, b)
        legendre_digit_sum_at(p, a + b)
        kummer_digit_sum_of_legendre_digit_sum(p, a, b)
        p * count_prime_factor(p, (a + b).binom(a)) + digit_sum(p, a + b) =
            count_prime_factor(p, (a + b).binom(a)) + digit_sum(p, a) + digit_sum(p, b)
    }
}
