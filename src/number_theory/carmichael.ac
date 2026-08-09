from number_theory.multiplicative_order import Nat, totient_positive,
    pow_mul_order_exponent_congr_one, multiplicative_order_mod,
    multiplicative_order_mod_minimal
from number_theory.totient import euler, totient_prime, divides_prime_imp_not_coprime
from number_theory.congruence import congr_mod_mul, congr_mod_trans,
    congr_mod_symm, congr_mod_pow, mod_congr_mod_self, mod_add_mul, mod_lt
from number_theory.coprime import coprime_comm, coprime_mod_iff, coprime_of_divisors,
    coprime_mul, coprime_divides_of_divides_mul, coprime_zero_left_imp_one
from number_theory.factorisation import no_proper_divisor_imp_prime,
    coprime_of_distinct_primes
from number_theory.fermat import prime_does_not_divide_below
from number_theory.zsigmondy import divides_suc_pair_imp_one, lt_ne
from number_theory.carmichael_residues import residue_eight_mod_two_one_odd
from nat import has_min, is_min, false_below, division_theorem, exp_add, exp_one,
    trichotomy, lte_imp_not_lt, lt_not_ref, not_lt_zero, small_mod,
    add_mod, add_imp_sub, gcd_zero_left, divides_self,
    lt_suc, lt_imp_lt_suc, lt_suc_right, lt_imp_lte_suc, lte_and_lt
numerals Nat

/// Positive exponents `m` for which `a^m ≡ 1 (mod n)` holds for every unit `a`
/// modulo `n`. The Carmichael function is the least such exponent.
define carmichael_witness(n: Nat) -> (Nat -> Bool) {
    function(m: Nat) {
        Nat.0 < m and forall(a: Nat) {
            a.coprime(n) implies a.pow(m).congr_mod(Nat.1, n)
        }
    }
}

/// Applying the witness predicate unfolds to positivity plus the universal
/// congruence.
theorem carmichael_witness_apply(n: Nat, m: Nat) {
    carmichael_witness(n)(m) =
        (Nat.0 < m and forall(a: Nat) {
            a.coprime(n) implies a.pow(m).congr_mod(Nat.1, n)
        })
}

/// From the witness predicate at `m` and a unit `a`, the congruence at `m`
/// follows.
theorem carmichael_witness_member(n: Nat, m: Nat, a: Nat) {
    carmichael_witness(n)(m) and a.coprime(n)
        implies a.pow(m).congr_mod(Nat.1, n)
} by {
    if carmichael_witness(n)(m) and a.coprime(n) {
        carmichael_witness_apply(n, m)
        carmichael_witness(n)(m) =
            (Nat.0 < m and forall(a0: Nat) {
                a0.coprime(n) implies a0.pow(m).congr_mod(Nat.1, n)
            })
        Nat.0 < m and forall(a0: Nat) {
            a0.coprime(n) implies a0.pow(m).congr_mod(Nat.1, n)
        }
        forall(a0: Nat) {
            a0.coprime(n) implies a0.pow(m).congr_mod(Nat.1, n)
        }
        a.coprime(n) implies a.pow(m).congr_mod(Nat.1, n)
        a.pow(m).congr_mod(Nat.1, n)
    }
}

/// True when `m` is the least positive universal exponent modulo `n`.
define is_carmichael(n: Nat, m: Nat) -> Bool {
    is_min(carmichael_witness(n), m)
}

/// If `1` is a universal exponent, then it is the least one.
theorem is_carmichael_one(n: Nat) {
    carmichael_witness(n)(Nat.1) implies is_carmichael(n, Nat.1)
} by {
    if carmichael_witness(n)(Nat.1) {
        // No positive `x < 1` exists, so nothing below `1` is a witness.
        forall(x: Nat) {
            if x < Nat.1 {
                lt_suc_right(x, Nat.0)
                x = Nat.0 or x < Nat.0
                if x < Nat.0 {
                    not_lt_zero(x)
                    false
                }
                x = Nat.0
                if carmichael_witness(n)(x) {
                    carmichael_witness_apply(n, x)
                    carmichael_witness(n)(x) =
                        (Nat.0 < x and forall(a0: Nat) {
                            a0.coprime(n) implies a0.pow(x).congr_mod(Nat.1, n)
                        })
                    Nat.0 < x
                    Nat.0 < Nat.0
                    lt_not_ref(Nat.0)
                    false
                }
                not carmichael_witness(n)(x)
            }
        }
        false_below(carmichael_witness(n), Nat.1)
        is_carmichael(n, Nat.1) = is_min(carmichael_witness(n), Nat.1)
        is_min(carmichael_witness(n), Nat.1) =
            (carmichael_witness(n)(Nat.1) and false_below(carmichael_witness(n), Nat.1))
        carmichael_witness(n)(Nat.1) and false_below(carmichael_witness(n), Nat.1)
        is_min(carmichael_witness(n), Nat.1)
        is_carmichael(n, Nat.1)
    }
}

/// If `2` is a universal exponent and `1` is not, then `2` is the least one.
theorem is_carmichael_two(n: Nat) {
    carmichael_witness(n)(Nat.2) and not carmichael_witness(n)(Nat.1)
        implies is_carmichael(n, Nat.2)
} by {
    if carmichael_witness(n)(Nat.2) and not carmichael_witness(n)(Nat.1) {
        forall(x: Nat) {
            if x < Nat.2 {
                lt_suc_right(x, Nat.1)
                if x = Nat.1 {
                    not carmichael_witness(n)(x)
                } else {
                    x < Nat.1
                    lt_suc_right(x, Nat.0)
                    x = Nat.0 or x < Nat.0
                    if x < Nat.0 {
                        not_lt_zero(x)
                        false
                    }
                    x = Nat.0
                    if carmichael_witness(n)(x) {
                        carmichael_witness_apply(n, x)
                        carmichael_witness(n)(x) =
                            (Nat.0 < x and forall(a0: Nat) {
                                a0.coprime(n) implies a0.pow(x).congr_mod(Nat.1, n)
                            })
                        Nat.0 < x
                        Nat.0 < Nat.0
                        lt_not_ref(Nat.0)
                        false
                    }
                    not carmichael_witness(n)(x)
                }
            }
        }
        false_below(carmichael_witness(n), Nat.2)
        is_carmichael(n, Nat.2) = is_min(carmichael_witness(n), Nat.2)
        is_min(carmichael_witness(n), Nat.2) =
            (carmichael_witness(n)(Nat.2) and false_below(carmichael_witness(n), Nat.2))
        carmichael_witness(n)(Nat.2) and false_below(carmichael_witness(n), Nat.2)
        is_min(carmichael_witness(n), Nat.2)
        is_carmichael(n, Nat.2)
    }
}

/// A nonzero natural is positive.
theorem ne_zero_imp_pos(a: Nat) {
    a != Nat.0 implies Nat.0 < a
} by {
    if a != Nat.0 {
        not_lt_zero(a)
        trichotomy(Nat.0, a)
        if Nat.0 < a {
            Nat.0 < a
        } else {
            a < Nat.0 or a = Nat.0
            if a < Nat.0 {
                not_lt_zero(a)
                false
            }
            a = Nat.0
            a != Nat.0
            false
        }
    }
}

/// If `g` has order `p - 1` modulo the prime `p`, no positive exponent below
/// `p - 1` is a universal exponent modulo `p`.
theorem no_witness_below_prime_at(p: Nat, g: Nat, x: Nat) {
    p.is_prime and g.coprime(p) and multiplicative_order_mod(g, p) = p - Nat.1
        and x < p - Nat.1
        implies not carmichael_witness(p)(x)
} by {
    if p.is_prime and g.coprime(p) and multiplicative_order_mod(g, p) = p - Nat.1
        and x < p - Nat.1 {
        if x = Nat.0 {
            if carmichael_witness(p)(x) {
                carmichael_witness_apply(p, x)
                carmichael_witness(p)(x) =
                    (Nat.0 < x and forall(a0: Nat) {
                        a0.coprime(p) implies a0.pow(x).congr_mod(Nat.1, p)
                    })
                Nat.0 < x
                Nat.0 < Nat.0
                lt_not_ref(Nat.0)
                false
            }
            not carmichael_witness(p)(x)
        } else {
            x != Nat.0
            ne_zero_imp_pos(x)
            Nat.0 < x
            if carmichael_witness(p)(x) {
                carmichael_witness_member(p, x, g)
                g.pow(x).congr_mod(Nat.1, p)
                Nat.1 < p
                p != Nat.0
                multiplicative_order_mod_minimal(g, p, x)
                multiplicative_order_mod(g, p) <= x
                multiplicative_order_mod(g, p) = p - Nat.1
                p - Nat.1 <= x
                lte_imp_not_lt(p - Nat.1, x)
                not x < p - Nat.1
                false
            }
            not carmichael_witness(p)(x)
        }
    }
}

/// If `g` generates the units modulo the prime `p`, then `p - 1` is the least
/// universal exponent modulo `p`.
theorem is_carmichael_prime(p: Nat, g: Nat) {
    p.is_prime and g.coprime(p) and multiplicative_order_mod(g, p) = p - Nat.1
        and carmichael_witness(p)(p - Nat.1)
        implies is_carmichael(p, p - Nat.1)
} by {
    if p.is_prime and g.coprime(p) and multiplicative_order_mod(g, p) = p - Nat.1
        and carmichael_witness(p)(p - Nat.1) {
        forall(x: Nat) {
            if x < p - Nat.1 {
                if x = Nat.0 {
                    if carmichael_witness(p)(x) {
                        carmichael_witness_apply(p, x)
                        carmichael_witness(p)(x) =
                            (Nat.0 < x and forall(a0: Nat) {
                                a0.coprime(p) implies a0.pow(x).congr_mod(Nat.1, p)
                            })
                        Nat.0 < x
                        Nat.0 < Nat.0
                        lt_not_ref(Nat.0)
                        false
                    }
                    not carmichael_witness(p)(x)
                } else {
                    no_witness_below_prime_at(p, g, x)
                    not carmichael_witness(p)(x)
                }
            }
        }
        false_below(carmichael_witness(p), p - Nat.1) =
            forall(x: Nat) { x < p - Nat.1 implies not carmichael_witness(p)(x) }
        false_below(carmichael_witness(p), p - Nat.1)
        is_carmichael(p, p - Nat.1) = is_min(carmichael_witness(p), p - Nat.1)
        is_min(carmichael_witness(p), p - Nat.1) =
            (carmichael_witness(p)(p - Nat.1) and
                false_below(carmichael_witness(p), p - Nat.1))
        carmichael_witness(p)(p - Nat.1) and
            false_below(carmichael_witness(p), p - Nat.1)
        is_min(carmichael_witness(p), p - Nat.1)
        is_carmichael(p, p - Nat.1)
    }
}

/// The Carmichael function `λ(n)`: the least positive exponent `m` such that
/// `a^m ≡ 1 (mod n)` for every `a` coprime to `n` — the exponent of the
/// multiplicative group of units modulo `n`. Convention: `λ(0) = 0`.
let carmichael(n: Nat) -> m: Nat satisfy {
    (n != Nat.0 and is_carmichael(n, m)) or (n = Nat.0 and m = Nat.0)
} by {
    if n != Nat.0 {
        // Euler's totient is a positive witness: Euler's theorem gives
        // `a^φ(n) ≡ 1 (mod n)` for every unit `a`.
        totient_positive(n)
        Nat.0 < n.totient
        forall(a: Nat) {
            if a.coprime(n) {
                euler(n, a)
                a.pow(n.totient).congr_mod(Nat.1, n)
            }
        }
        carmichael_witness_apply(n, n.totient)
        carmichael_witness(n)(n.totient) =
            (Nat.0 < n.totient and forall(a: Nat) {
                a.coprime(n) implies a.pow(n.totient).congr_mod(Nat.1, n)
            })
        Nat.0 < n.totient and forall(a: Nat) {
            a.coprime(n) implies a.pow(n.totient).congr_mod(Nat.1, n)
        }
        carmichael_witness(n)(n.totient)
        has_min(carmichael_witness(n), n.totient)
        let m: Nat satisfy { is_min(carmichael_witness(n), m) }
        is_min(carmichael_witness(n), m)
        is_carmichael(n, m) = is_min(carmichael_witness(n), m)
        is_carmichael(n, m)
        n != Nat.0 and is_carmichael(n, m)
        (n != Nat.0 and is_carmichael(n, m)) or (n = Nat.0 and m = Nat.0)
    } else {
        n = Nat.0
        Nat.0 = Nat.0
        n = Nat.0 and Nat.0 = Nat.0
        (n != Nat.0 and is_carmichael(n, Nat.0)) or (n = Nat.0 and Nat.0 = Nat.0)
    }
}

/// Characterisation of the selected value, including the zero convention.
theorem carmichael_spec(n: Nat) {
    (n != Nat.0 and is_carmichael(n, carmichael(n)))
        or (n = Nat.0 and carmichael(n) = Nat.0)
}

/// On the positive domain, the selected value is the least witness.
theorem carmichael_is_carmichael(n: Nat) {
    n != Nat.0 implies is_carmichael(n, carmichael(n))
} by {
    if n != Nat.0 {
        carmichael_spec(n)
        if n != Nat.0 and is_carmichael(n, carmichael(n)) {
            is_carmichael(n, carmichael(n))
        } else {
            n = Nat.0 and carmichael(n) = Nat.0
            n = Nat.0
            false
        }
    }
}

/// `λ(n)` is positive for every positive modulus.
theorem carmichael_positive(n: Nat) {
    n != Nat.0 implies Nat.0 < carmichael(n)
} by {
    if n != Nat.0 {
        carmichael_is_carmichael(n)
        is_carmichael(n, carmichael(n))
        is_carmichael(n, carmichael(n)) = is_min(carmichael_witness(n), carmichael(n))
        is_min(carmichael_witness(n), carmichael(n))
        is_min(carmichael_witness(n), carmichael(n)) =
            (carmichael_witness(n)(carmichael(n)) and
                false_below(carmichael_witness(n), carmichael(n)))
        carmichael_witness(n)(carmichael(n)) and
            false_below(carmichael_witness(n), carmichael(n))
        carmichael_witness(n)(carmichael(n))
        carmichael_witness_apply(n, carmichael(n))
        carmichael_witness(n)(carmichael(n)) =
            (Nat.0 < carmichael(n) and forall(a: Nat) {
                a.coprime(n) implies a.pow(carmichael(n)).congr_mod(Nat.1, n)
            })
        Nat.0 < carmichael(n) and forall(a: Nat) {
            a.coprime(n) implies a.pow(carmichael(n)).congr_mod(Nat.1, n)
        }
        Nat.0 < carmichael(n)
    }
}

/// Euler's theorem refines to `λ(n)`: for `n != 0` and `a` coprime to `n`,
/// `a^λ(n) ≡ 1 (mod n)`.
theorem carmichael_pow_congr_one(n: Nat, a: Nat) {
    n != Nat.0 and a.coprime(n)
        implies a.pow(carmichael(n)).congr_mod(Nat.1, n)
} by {
    if n != Nat.0 and a.coprime(n) {
        carmichael_is_carmichael(n)
        is_carmichael(n, carmichael(n))
        is_carmichael(n, carmichael(n)) = is_min(carmichael_witness(n), carmichael(n))
        is_min(carmichael_witness(n), carmichael(n))
        is_min(carmichael_witness(n), carmichael(n)) =
            (carmichael_witness(n)(carmichael(n)) and
                false_below(carmichael_witness(n), carmichael(n)))
        carmichael_witness(n)(carmichael(n)) and
            false_below(carmichael_witness(n), carmichael(n))
        carmichael_witness(n)(carmichael(n))
        carmichael_witness_member(n, carmichael(n), a)
        a.pow(carmichael(n)).congr_mod(Nat.1, n)
    }
}

/// If `k` is a witness and `m` is a least witness, then `m <= k`.
theorem carmichael_least_witness_le(n: Nat, m: Nat, k: Nat) {
    is_min(carmichael_witness(n), m) and carmichael_witness(n)(k)
        implies m <= k
} by {
    if is_min(carmichael_witness(n), m) and carmichael_witness(n)(k) {
        is_min(carmichael_witness(n), m) =
            (carmichael_witness(n)(m) and false_below(carmichael_witness(n), m))
        carmichael_witness(n)(m) and false_below(carmichael_witness(n), m)
        false_below(carmichael_witness(n), m)
        if k < m {
            false_below(carmichael_witness(n), m) = forall(x: Nat) {
                x < m implies not carmichael_witness(n)(x)
            }
            not carmichael_witness(n)(k)
            false
        } else {
            m <= k
        }
        m <= k
    }
}

/// Minimality: any positive universal exponent is at least `λ(n)`.
theorem carmichael_minimal(n: Nat, m: Nat) {
    n != Nat.0 and carmichael_witness(n)(m) implies carmichael(n) <= m
} by {
    if n != Nat.0 and carmichael_witness(n)(m) {
        carmichael_is_carmichael(n)
        is_carmichael(n, carmichael(n))
        is_carmichael(n, carmichael(n)) = is_min(carmichael_witness(n), carmichael(n))
        is_min(carmichael_witness(n), carmichael(n))
        carmichael_least_witness_le(n, carmichael(n), m)
        carmichael(n) <= m
    }
}

/// An explicit least witness determines `λ(n)` uniquely.
theorem carmichael_eq_of_is_carmichael(n: Nat, m: Nat) {
    n != Nat.0 and is_carmichael(n, m) implies carmichael(n) = m
} by {
    if n != Nat.0 and is_carmichael(n, m) {
        is_carmichael(n, m) = is_min(carmichael_witness(n), m)
        is_min(carmichael_witness(n), m)
        is_min(carmichael_witness(n), m) =
            (carmichael_witness(n)(m) and false_below(carmichael_witness(n), m))
        carmichael_witness(n)(m) and false_below(carmichael_witness(n), m)
        carmichael_witness(n)(m)
        carmichael_minimal(n, m)
        carmichael(n) <= m
        carmichael_is_carmichael(n)
        is_carmichael(n, carmichael(n))
        is_carmichael(n, carmichael(n)) = is_min(carmichael_witness(n), carmichael(n))
        is_min(carmichael_witness(n), carmichael(n))
        carmichael_least_witness_le(n, m, carmichael(n))
        m <= carmichael(n)
        carmichael(n) = m
    }
}

/// `λ(n)` divides Euler's totient `φ(n)` for every positive modulus: since
/// `φ(n)` is itself a universal exponent (Euler), the least such exponent
/// divides it.
theorem carmichael_divides_totient(n: Nat) {
    n != Nat.0 implies carmichael(n).divides(n.totient)
} by {
    if n != Nat.0 {
        carmichael_positive(n)
        Nat.0 < carmichael(n)
        division_theorem(n.totient, carmichael(n))
        let (q: Nat, r: Nat) satisfy {
            r < carmichael(n) and n.totient = q * carmichael(n) + r
        }
        r < carmichael(n)
        n.totient = q * carmichael(n) + r
        // The remainder `r` is again a universal exponent: `a^r ≡ 1` for
        // every unit `a`.
        forall(a: Nat) {
            if a.coprime(n) {
                carmichael_pow_congr_one(n, a)
                a.pow(carmichael(n)).congr_mod(Nat.1, n)
                pow_mul_order_exponent_congr_one(a, n, carmichael(n), q)
                a.pow(q * carmichael(n)).congr_mod(Nat.1, n)
                euler(n, a)
                a.pow(n.totient).congr_mod(Nat.1, n)
                n.totient = q * carmichael(n) + r
                a.pow(n.totient) = a.pow(q * carmichael(n) + r)
                exp_add(a, q * carmichael(n), r)
                a.pow(q * carmichael(n) + r) =
                    a.pow(q * carmichael(n)) * a.pow(r)
                a.pow(n.totient) = a.pow(q * carmichael(n)) * a.pow(r)
                (a.pow(q * carmichael(n)) * a.pow(r)).congr_mod(Nat.1, n)
                congr_mod_mul(a.pow(q * carmichael(n)), a.pow(r),
                    Nat.1, a.pow(r), n)
                (a.pow(q * carmichael(n)) * a.pow(r)).congr_mod(
                    Nat.1 * a.pow(r), n)
                Nat.1 * a.pow(r) = a.pow(r)
                (a.pow(q * carmichael(n)) * a.pow(r)).congr_mod(a.pow(r), n)
                congr_mod_symm(a.pow(q * carmichael(n)) * a.pow(r), Nat.1, n)
                Nat.1.congr_mod(a.pow(q * carmichael(n)) * a.pow(r), n)
                congr_mod_trans(Nat.1,
                    a.pow(q * carmichael(n)) * a.pow(r), a.pow(r), n)
                Nat.1.congr_mod(a.pow(r), n)
                congr_mod_symm(Nat.1, a.pow(r), n)
                a.pow(r).congr_mod(Nat.1, n)
            }
        }
        // A positive remainder would contradict the minimality of `λ(n)`.
        if Nat.0 < r {
            carmichael_witness_apply(n, r)
            carmichael_witness(n)(r) =
                (Nat.0 < r and forall(a: Nat) {
                    a.coprime(n) implies a.pow(r).congr_mod(Nat.1, n)
                })
            Nat.0 < r and forall(a: Nat) {
                a.coprime(n) implies a.pow(r).congr_mod(Nat.1, n)
            }
            carmichael_witness(n)(r)
            carmichael_minimal(n, r)
            carmichael(n) <= r
            lte_imp_not_lt(carmichael(n), r)
            not r < carmichael(n)
            false
        }
        not Nat.0 < r
        trichotomy(Nat.0, r)
        r = Nat.0
        n.totient = q * carmichael(n) + r
        q * carmichael(n) + Nat.0 = q * carmichael(n)
        n.totient = q * carmichael(n)
        q * carmichael(n) = carmichael(n) * q
        carmichael(n) * q = n.totient
        carmichael(n).divides(n.totient)
    }
}

/// For a prime modulus, `λ(p)` divides `p - 1`, since `φ(p) = p - 1`.
theorem carmichael_divides_prime_minus_one(p: Nat) {
    p.is_prime implies carmichael(p).divides(p - Nat.1)
} by {
    if p.is_prime {
        Nat.1 < p
        p != Nat.0
        carmichael_divides_totient(p)
        carmichael(p).divides(p.totient)
        totient_prime(p)
        p.totient = p - Nat.1
        carmichael(p).divides(p - Nat.1)
    }
}

/// If `g` is a primitive root modulo the prime `p` (an element of order
/// `p - 1`), then `λ(p) = p - 1`.
theorem carmichael_prime_of_generator(p: Nat, g: Nat) {
    p.is_prime and g.coprime(p) and multiplicative_order_mod(g, p) = p - Nat.1
        implies carmichael(p) = p - Nat.1
} by {
    if p.is_prime and g.coprime(p) and multiplicative_order_mod(g, p) = p - Nat.1 {
        // `p - 1` is a universal exponent: Fermat's theorem.
        Nat.1 < p
        p != Nat.0
        forall(a: Nat) {
            if a.coprime(p) {
                euler(p, a)
                a.pow(p.totient).congr_mod(Nat.1, p)
                totient_prime(p)
                p.totient = p - Nat.1
                a.pow(p - Nat.1).congr_mod(Nat.1, p)
            }
        }
        totient_positive(p)
        Nat.0 < p.totient
        p.totient = p - Nat.1
        Nat.0 < p - Nat.1
        carmichael_witness_apply(p, p - Nat.1)
        carmichael_witness(p)(p - Nat.1) =
            (Nat.0 < p - Nat.1 and forall(a: Nat) {
                a.coprime(p) implies a.pow(p - Nat.1).congr_mod(Nat.1, p)
            })
        Nat.0 < p - Nat.1 and forall(a: Nat) {
            a.coprime(p) implies a.pow(p - Nat.1).congr_mod(Nat.1, p)
        }
        carmichael_witness(p)(p - Nat.1)
        // No smaller positive exponent is universal: `g` has order `p - 1`.
        forall(j: Nat) {
            if Nat.0 < j and j < p - Nat.1 {
                if carmichael_witness(p)(j) {
                    carmichael_witness_member(p, j, g)
                    g.pow(j).congr_mod(Nat.1, p)
                    multiplicative_order_mod_minimal(g, p, j)
                    multiplicative_order_mod(g, p) <= j
                    multiplicative_order_mod(g, p) = p - Nat.1
                    p - Nat.1 <= j
                    lte_imp_not_lt(p - Nat.1, j)
                    not j < p - Nat.1
                    false
                }
                not carmichael_witness(p)(j)
            }
        }
        is_carmichael_prime(p, g)
        is_carmichael(p, p - Nat.1)
        carmichael_eq_of_is_carmichael(p, p - Nat.1)
        carmichael(p) = p - Nat.1
    }
}

// ---------------------------------------------------------------------------
// ---------------------------------------------------------------------------
// Small values of the Carmichael function.
// ---------------------------------------------------------------------------

/// `1 < 2`.
theorem lt_one_two {
    Nat.1 < Nat.2
} by {
    lt_suc(Nat.1)
}

/// `2 < 3`.
theorem lt_two_three {
    Nat.2 < Nat.3
} by {
    lt_suc(Nat.2)
}

/// `3 < 4`.
theorem lt_three_four {
    Nat.3 < Nat.4
} by {
    lt_suc(Nat.3)
}

/// `1 < 3`.
theorem lt_one_three {
    Nat.1 < Nat.3
} by {
    lt_suc(Nat.1)
    Nat.1 < Nat.2
    lt_imp_lt_suc(Nat.1, Nat.2)
    Nat.1 < Nat.3
}

/// `1 < 4`.
theorem lt_one_four {
    Nat.1 < Nat.4
} by {
    lt_suc(Nat.1)
    Nat.1 < Nat.2
    lt_imp_lt_suc(Nat.1, Nat.2)
    Nat.1 < Nat.3
    lt_imp_lt_suc(Nat.1, Nat.3)
    Nat.1 < Nat.4
}

/// `0 < 2`.
theorem lt_zero_two {
    Nat.0 < Nat.2
} by {
    lt_suc(Nat.0)
    Nat.0 < Nat.1
    lt_imp_lt_suc(Nat.0, Nat.1)
    Nat.0 < Nat.2
}

/// `1 < 8`.
theorem lt_one_eight {
    Nat.1 < Nat.8
} by {
    lt_one_four
    Nat.1 < Nat.4
    lt_imp_lt_suc(Nat.1, Nat.4)
    Nat.1 < Nat.5
    lt_imp_lt_suc(Nat.1, Nat.5)
    Nat.1 < Nat.6
    lt_imp_lt_suc(Nat.1, Nat.6)
    Nat.1 < Nat.7
    lt_imp_lt_suc(Nat.1, Nat.7)
    Nat.1 < Nat.8
}

/// `3 < 8`.
theorem lt_three_eight {
    Nat.3 < Nat.8
} by {
    lt_three_four
    Nat.3 < Nat.4
    lt_imp_lt_suc(Nat.3, Nat.4)
    Nat.3 < Nat.5
    lt_imp_lt_suc(Nat.3, Nat.5)
    Nat.3 < Nat.6
    lt_imp_lt_suc(Nat.3, Nat.6)
    Nat.3 < Nat.7
    lt_imp_lt_suc(Nat.3, Nat.7)
    Nat.3 < Nat.8
}

/// `1 != 0`.
theorem one_ne_zero {
    Nat.1 != Nat.0
}


/// `2 != 1`.
theorem two_ne_one {
    Nat.2 != Nat.1
} by {
    lt_ne(Nat.1, Nat.2, Nat.1)
    Nat.1 + Nat.1 = Nat.2
    one_ne_zero
}

/// `3 != 2`.
theorem three_ne_two {
    Nat.3 != Nat.2
} by {
    lt_ne(Nat.2, Nat.3, Nat.1)
    Nat.2 + Nat.1 = Nat.3
    one_ne_zero
}

/// `3 != 1`.
theorem three_ne_one {
    Nat.3 != Nat.1
} by {
    lt_ne(Nat.1, Nat.3, Nat.2)
    Nat.1 + Nat.2 = Nat.3
    Nat.2 != Nat.0
}

/// `1.mod(2) = 1`.
theorem one_mod_two {
    Nat.1.mod(Nat.2) = Nat.1
} by {
    lt_one_two
    Nat.1 < Nat.2
    small_mod(Nat.1, Nat.2)
    Nat.1.mod(Nat.2) = Nat.1
}

/// `1.mod(4) = 1`.
theorem one_mod_four {
    Nat.1.mod(Nat.4) = Nat.1
} by {
    lt_one_four
    Nat.1 < Nat.4
    small_mod(Nat.1, Nat.4)
    Nat.1.mod(Nat.4) = Nat.1
}

/// `3.mod(4) = 3`.
theorem three_mod_four {
    Nat.3.mod(Nat.4) = Nat.3
} by {
    lt_three_four
    Nat.3 < Nat.4
    small_mod(Nat.3, Nat.4)
    Nat.3.mod(Nat.4) = Nat.3
}

/// `1.mod(8) = 1`.
theorem one_mod_eight {
    Nat.1.mod(Nat.8) = Nat.1
} by {
    lt_one_eight
    Nat.1 < Nat.8
    small_mod(Nat.1, Nat.8)
    Nat.1.mod(Nat.8) = Nat.1
}

/// `3.mod(8) = 3`.
theorem three_mod_eight {
    Nat.3.mod(Nat.8) = Nat.3
} by {
    lt_three_eight
    Nat.3 < Nat.8
    small_mod(Nat.3, Nat.8)
    Nat.3.mod(Nat.8) = Nat.3
}

/// A number below `2` that is not zero is one.
theorem below_two_not_zero_imp_one(a: Nat) {
    a < Nat.2 and a != Nat.0 implies a = Nat.1
} by {
    if a < Nat.2 and a != Nat.0 {
        if a = Nat.1 {
            a = Nat.1
        } else {
            a != Nat.1
            lt_suc_right(a, Nat.1)
            a < Nat.1
            lt_suc_right(a, Nat.0)
            a = Nat.0 or a < Nat.0
            if a < Nat.0 {
                not_lt_zero(a)
                false
            }
            a = Nat.0
            a != Nat.0
            false
        }
    }
}

/// Every unit modulo `2` is congruent to `1` modulo `2`.
theorem coprime_two_congr_one(a: Nat) {
    a.coprime(Nat.2) implies a.congr_mod(Nat.1, Nat.2)
} by {
    if a.coprime(Nat.2) {
        coprime_mod_iff(a, Nat.2)
        a.coprime(Nat.2) = a.mod(Nat.2).coprime(Nat.2)
        a.mod(Nat.2).coprime(Nat.2)
        mod_lt(a, Nat.2)
        a.mod(Nat.2) < Nat.2
        if a.mod(Nat.2) = Nat.0 {
            Nat.0.coprime(Nat.2)
            gcd_zero_left(Nat.2)
            Nat.0.gcd(Nat.2) = Nat.2
            Nat.2 = Nat.1
            two_ne_one
            false
        }
        a.mod(Nat.2) != Nat.0
        below_two_not_zero_imp_one(a.mod(Nat.2))
        a.mod(Nat.2) = Nat.1
        one_mod_two
        Nat.1.mod(Nat.2) = Nat.1
        a.mod(Nat.2) = Nat.1.mod(Nat.2)
        a.congr_mod(Nat.1, Nat.2)
    }
}

/// The square of a natural is the product with itself.
theorem pow_two(a: Nat) {
    a.pow(Nat.2) = a * a
} by {
    exp_add(a, Nat.1, Nat.1)
    a.pow(Nat.1 + Nat.1) = a.pow(Nat.1) * a.pow(Nat.1)
    Nat.1 + Nat.1 = Nat.2
    a.pow(Nat.2) = a.pow(Nat.1) * a.pow(Nat.1)
    exp_one(a)
    a.pow(Nat.1) = a
    a.pow(Nat.2) = a * a
}

/// Two is prime.
theorem two_is_prime {
    Nat.2.is_prime
} by {
    lt_one_two
    Nat.1 < Nat.2
    forall(k: Nat) {
        if Nat.1 < k and k < Nat.2 {
            lt_imp_lte_suc(Nat.1, k)
            Nat.2 <= k
            lte_and_lt(Nat.2, k, Nat.2)
            Nat.2 < Nat.2
            lt_not_ref(Nat.2)
            false
        }
    }
    no_proper_divisor_imp_prime(Nat.2)
}

/// `4 != 1`.
theorem four_ne_one {
    Nat.4 != Nat.1
} by {
    lt_one_four
    if Nat.4 = Nat.1 {
        Nat.1 < Nat.1
        lt_not_ref(Nat.1)
        false
    }
}

/// Zero is not a unit modulo `4`.
theorem not_zero_coprime_four {
    not Nat.0.coprime(Nat.4)
} by {
    if Nat.0.coprime(Nat.4) {
        coprime_zero_left_imp_one(Nat.4)
        Nat.4 = Nat.1
        four_ne_one
        false
    }
}

/// Two is not a unit modulo `4`.
theorem not_two_coprime_four {
    not Nat.2.coprime(Nat.4)
} by {
    if Nat.2.coprime(Nat.4) {
        Nat.2 * Nat.2 = Nat.4
        exists(c: Nat) { Nat.2 * c = Nat.4 }
        Nat.2.divides(Nat.4)
        divides_prime_imp_not_coprime(Nat.2, Nat.4)
        two_is_prime
        not Nat.4.coprime(Nat.2)
        coprime_comm(Nat.2, Nat.4)
        Nat.4.coprime(Nat.2)
        false
    }
}

/// A residue below `4` that is coprime to `4` is `1` or `3`.
theorem residue_four_coprime_one_or_three(r: Nat) {
    r < Nat.4 and r.coprime(Nat.4) implies (r = Nat.1 or r = Nat.3)
} by {
    if r < Nat.4 and r.coprime(Nat.4) {
        lt_suc_right(r, Nat.3)
        if r = Nat.3 {
            r = Nat.1 or r = Nat.3
        } else {
            r < Nat.3
            lt_suc_right(r, Nat.2)
            if r = Nat.2 {
                Nat.2.coprime(Nat.4)
                not_two_coprime_four
                false
            } else {
                r < Nat.2
                lt_suc_right(r, Nat.1)
                if r = Nat.1 {
                    r = Nat.1 or r = Nat.3
                } else {
                    r < Nat.1
                    lt_suc_right(r, Nat.0)
                    r = Nat.0 or r < Nat.0
                    if r < Nat.0 {
                        not_lt_zero(r)
                        false
                    }
                    r = Nat.0
                    Nat.0.coprime(Nat.4)
                    not_zero_coprime_four
                    false
                }
            }
        }
    }
}

/// A residue that is `1` or `3` squares to `1` modulo `4`.
theorem residue_four_square_one(r: Nat) {
    (r = Nat.1 or r = Nat.3) implies r.pow(Nat.2).congr_mod(Nat.1, Nat.4)
} by {
    if r = Nat.1 or r = Nat.3 {
        if r = Nat.1 {
            pow_two(r)
            r.pow(Nat.2) = r * r
            r * r = Nat.1 * Nat.1
            Nat.1 * Nat.1 = Nat.1
            r.pow(Nat.2) = Nat.1
            one_mod_four
            Nat.1.mod(Nat.4) = Nat.1
            r.pow(Nat.2).mod(Nat.4) = Nat.1.mod(Nat.4)
            r.pow(Nat.2).congr_mod(Nat.1, Nat.4)
        } else {
            r = Nat.3
            pow_two(r)
            r.pow(Nat.2) = r * r
            r * r = Nat.3 * Nat.3
            Nat.3 * Nat.3 = Nat.9
            r.pow(Nat.2) = Nat.9
            // `9 ≡ 1 (mod 4)`.
            Nat.2 * Nat.4 + Nat.1 = Nat.9
            mod_add_mul(Nat.2, Nat.4, Nat.1)
            (Nat.2 * Nat.4 + Nat.1).mod(Nat.4) = Nat.1.mod(Nat.4)
            Nat.9.mod(Nat.4) = Nat.1.mod(Nat.4)
            one_mod_four
            Nat.1.mod(Nat.4) = Nat.1
            Nat.9.mod(Nat.4) = Nat.1
            Nat.9.congr_mod(Nat.1, Nat.4)
            r.pow(Nat.2).congr_mod(Nat.9, Nat.4)
            congr_mod_trans(r.pow(Nat.2), Nat.9, Nat.1, Nat.4)
            r.pow(Nat.2).congr_mod(Nat.1, Nat.4)
        }
    }
}

/// Every unit modulo `4` squares to `1` modulo `4`.
theorem unit_four_square_one(a: Nat) {
    a.coprime(Nat.4) implies a.pow(Nat.2).congr_mod(Nat.1, Nat.4)
} by {
    if a.coprime(Nat.4) {
        coprime_mod_iff(a, Nat.4)
        a.mod(Nat.4).coprime(Nat.4)
        mod_lt(a, Nat.4)
        a.mod(Nat.4) < Nat.4
        residue_four_coprime_one_or_three(a.mod(Nat.4))
        a.mod(Nat.4) = Nat.1 or a.mod(Nat.4) = Nat.3
        residue_four_square_one(a.mod(Nat.4))
        a.mod(Nat.4).pow(Nat.2).congr_mod(Nat.1, Nat.4)
        mod_congr_mod_self(a, Nat.4)
        a.mod(Nat.4).congr_mod(a, Nat.4)
        congr_mod_symm(a.mod(Nat.4), a, Nat.4)
        a.congr_mod(a.mod(Nat.4), Nat.4)
        congr_mod_pow(a, a.mod(Nat.4), Nat.4, Nat.2)
        a.pow(Nat.2).congr_mod(a.mod(Nat.4).pow(Nat.2), Nat.4)
        congr_mod_trans(a.pow(Nat.2), a.mod(Nat.4).pow(Nat.2), Nat.1, Nat.4)
        a.pow(Nat.2).congr_mod(Nat.1, Nat.4)
    }
}

/// An odd residue `1`, `3`, `5` or `7` squares to `1` modulo `8`.
theorem residue_eight_square_one(r: Nat) {
    (r = Nat.1 or r = Nat.3 or r = Nat.5 or r = Nat.7)
        implies r.pow(Nat.2).congr_mod(Nat.1, Nat.8)
} by {
    if r = Nat.1 or r = Nat.3 or r = Nat.5 or r = Nat.7 {
        if r = Nat.1 {
            pow_two(r)
            r.pow(Nat.2) = r * r
            r * r = Nat.1 * Nat.1
            Nat.1 * Nat.1 = Nat.1
            r.pow(Nat.2) = Nat.1
            one_mod_eight
            Nat.1.mod(Nat.8) = Nat.1
            r.pow(Nat.2).mod(Nat.8) = Nat.1.mod(Nat.8)
            r.pow(Nat.2).congr_mod(Nat.1, Nat.8)
        } else {
            if r = Nat.3 {
                pow_two(r)
                r.pow(Nat.2) = r * r
                r * r = Nat.3 * Nat.3
                Nat.3 * Nat.3 = Nat.9
                r.pow(Nat.2) = Nat.9
                // `9 ≡ 1 (mod 8)`.
                Nat.1 * Nat.8 + Nat.1 = Nat.9
                mod_add_mul(Nat.1, Nat.8, Nat.1)
                (Nat.1 * Nat.8 + Nat.1).mod(Nat.8) = Nat.1.mod(Nat.8)
                Nat.9.mod(Nat.8) = Nat.1.mod(Nat.8)
                one_mod_eight
                Nat.1.mod(Nat.8) = Nat.1
                Nat.9.mod(Nat.8) = Nat.1
                Nat.9.congr_mod(Nat.1, Nat.8)
                r.pow(Nat.2).congr_mod(Nat.9, Nat.8)
                congr_mod_trans(r.pow(Nat.2), Nat.9, Nat.1, Nat.8)
                r.pow(Nat.2).congr_mod(Nat.1, Nat.8)
            } else {
                if r = Nat.5 {
                    pow_two(r)
                    r.pow(Nat.2) = r * r
                    r * r = Nat.5 * Nat.5
                    Nat.5 * Nat.5 = Nat.25
                    r.pow(Nat.2) = Nat.25
                    // `25 ≡ 1 (mod 8)`.
                    Nat.3 * Nat.8 = Nat.24
                    Nat.24 + Nat.1 = Nat.25
                    Nat.3 * Nat.8 + Nat.1 = Nat.25
                    mod_add_mul(Nat.3, Nat.8, Nat.1)
                    (Nat.3 * Nat.8 + Nat.1).mod(Nat.8) = Nat.1.mod(Nat.8)
                    Nat.25.mod(Nat.8) = Nat.1.mod(Nat.8)
                    one_mod_eight
                    Nat.1.mod(Nat.8) = Nat.1
                    Nat.25.mod(Nat.8) = Nat.1
                    Nat.25.congr_mod(Nat.1, Nat.8)
                    r.pow(Nat.2).congr_mod(Nat.25, Nat.8)
                    congr_mod_trans(r.pow(Nat.2), Nat.25, Nat.1, Nat.8)
                    r.pow(Nat.2).congr_mod(Nat.1, Nat.8)
                } else {
                    r = Nat.7
                    pow_two(r)
                    r.pow(Nat.2) = r * r
                    r * r = Nat.7 * Nat.7
                    Nat.7 * Nat.7 = Nat.49
                    r.pow(Nat.2) = Nat.49
                    // `49 ≡ 1 (mod 8)`.
                    Nat.6 * Nat.8 = Nat.48
                    Nat.48 + Nat.1 = Nat.49
                    Nat.6 * Nat.8 + Nat.1 = Nat.49
                    mod_add_mul(Nat.6, Nat.8, Nat.1)
                    (Nat.6 * Nat.8 + Nat.1).mod(Nat.8) = Nat.1.mod(Nat.8)
                    Nat.49.mod(Nat.8) = Nat.1.mod(Nat.8)
                    one_mod_eight
                    Nat.1.mod(Nat.8) = Nat.1
                    Nat.49.mod(Nat.8) = Nat.1
                    Nat.49.congr_mod(Nat.1, Nat.8)
                    r.pow(Nat.2).congr_mod(Nat.49, Nat.8)
                    congr_mod_trans(r.pow(Nat.2), Nat.49, Nat.1, Nat.8)
                    r.pow(Nat.2).congr_mod(Nat.1, Nat.8)
                }
            }
        }
    }
}

/// Every unit modulo `8` squares to `1` modulo `8`.
theorem unit_eight_square_one(a: Nat) {
    a.coprime(Nat.8) implies a.pow(Nat.2).congr_mod(Nat.1, Nat.8)
} by {
    if a.coprime(Nat.8) {
        coprime_mod_iff(a, Nat.8)
        a.mod(Nat.8).coprime(Nat.8)
        mod_lt(a, Nat.8)
        a.mod(Nat.8) < Nat.8
        coprime_of_divisors(a.mod(Nat.8), Nat.8, a.mod(Nat.8), Nat.2)
        divides_self(a.mod(Nat.8))
        Nat.2.divides(Nat.8)
        a.mod(Nat.8).coprime(Nat.2)
        coprime_two_congr_one(a.mod(Nat.8))
        a.mod(Nat.8).congr_mod(Nat.1, Nat.2)
        a.mod(Nat.8).mod(Nat.2) = Nat.1.mod(Nat.2)
        one_mod_two
        Nat.1.mod(Nat.2) = Nat.1
        a.mod(Nat.8).mod(Nat.2) = Nat.1
        residue_eight_mod_two_one_odd(a.mod(Nat.8))
        a.mod(Nat.8) = Nat.1 or a.mod(Nat.8) = Nat.3 or a.mod(Nat.8) = Nat.5 or a.mod(Nat.8) = Nat.7
        residue_eight_square_one(a.mod(Nat.8))
        a.mod(Nat.8).pow(Nat.2).congr_mod(Nat.1, Nat.8)
        mod_congr_mod_self(a, Nat.8)
        a.mod(Nat.8).congr_mod(a, Nat.8)
        congr_mod_symm(a.mod(Nat.8), a, Nat.8)
        a.congr_mod(a.mod(Nat.8), Nat.8)
        congr_mod_pow(a, a.mod(Nat.8), Nat.8, Nat.2)
        a.pow(Nat.2).congr_mod(a.mod(Nat.8).pow(Nat.2), Nat.8)
        congr_mod_trans(a.pow(Nat.2), a.mod(Nat.8).pow(Nat.2), Nat.1, Nat.8)
        a.pow(Nat.2).congr_mod(Nat.1, Nat.8)
    }
}

/// Two does not divide three.
theorem not_two_divides_three {
    not Nat.2.divides(Nat.3)
} by {
    if Nat.2.divides(Nat.3) {
        Nat.2 * Nat.1 = Nat.2
        exists(c: Nat) { Nat.2 * c = Nat.2 }
        Nat.2.divides(Nat.2)
        divides_suc_pair_imp_one(Nat.2, Nat.2)
        Nat.2 = Nat.1
        two_ne_one
        false
    }
}

/// No number strictly between one and three divides three.
theorem three_no_proper_divisor(k: Nat) {
    Nat.1 < k and k < Nat.3 implies not k.divides(Nat.3)
} by {
    if Nat.1 < k and k < Nat.3 {
        lt_suc_right(k, Nat.2)
        k = Nat.2 or k < Nat.2
        if k < Nat.2 {
            lt_imp_lte_suc(Nat.1, k)
            Nat.2 <= k
            lte_and_lt(Nat.2, k, Nat.2)
            Nat.2 < Nat.2
            lt_not_ref(Nat.2)
            false
        } else {
            k = Nat.2
            not_two_divides_three
            not Nat.2.divides(Nat.3)
            not k.divides(Nat.3)
        }
    }
}

/// Three is prime.
theorem three_is_prime {
    Nat.3.is_prime
} by {
    lt_one_three
    Nat.1 < Nat.3
    forall(k: Nat) {
        three_no_proper_divisor(k)
    }
    no_proper_divisor_imp_prime(Nat.3)
}

/// Three does not divide two.
theorem not_three_divides_two {
    not Nat.3.divides(Nat.2)
} by {
    three_is_prime
    Nat.3.is_prime
    lt_zero_two
    Nat.0 < Nat.2
    lt_two_three
    Nat.2 < Nat.3
    prime_does_not_divide_below(Nat.3, Nat.2)
    not Nat.3.divides(Nat.2)
}

/// Three does not divide four.
theorem not_three_divides_four {
    not Nat.3.divides(Nat.4)
} by {
    coprime_of_distinct_primes(Nat.3, Nat.2)
    Nat.3.coprime(Nat.2)
    Nat.4 = Nat.2 * Nat.2
    if Nat.3.divides(Nat.4) {
        Nat.3.divides(Nat.2 * Nat.2)
        coprime_divides_of_divides_mul(Nat.3, Nat.2, Nat.2)
        Nat.3.divides(Nat.2)
        not_three_divides_two
        false
    }
}

/// The unit `3` modulo `4`.
theorem three_coprime_four {
    Nat.3.coprime(Nat.4)
} by {
    coprime_of_distinct_primes(Nat.3, Nat.2)
    Nat.3.coprime(Nat.2)
    coprime_mul(Nat.3, Nat.2, Nat.2)
    Nat.3.coprime(Nat.2 * Nat.2)
    Nat.4 = Nat.2 * Nat.2
    Nat.3.coprime(Nat.4)
}

/// The unit `3` modulo `8`.
theorem three_coprime_eight {
    Nat.3.coprime(Nat.8)
} by {
    coprime_of_distinct_primes(Nat.3, Nat.2)
    Nat.3.coprime(Nat.2)
    coprime_mul(Nat.3, Nat.2, Nat.2)
    Nat.3.coprime(Nat.2 * Nat.2)
    Nat.4 = Nat.2 * Nat.2
    Nat.3.coprime(Nat.4)
    coprime_mul(Nat.3, Nat.4, Nat.2)
    Nat.3.coprime(Nat.4 * Nat.2)
    Nat.4 * Nat.2 = Nat.8
    Nat.3.coprime(Nat.8)
}

/// `3 ≢ 1 (mod 4)`.
theorem not_three_congr_one_mod_four {
    not Nat.3.congr_mod(Nat.1, Nat.4)
} by {
    three_mod_four
    Nat.3.mod(Nat.4) = Nat.3
    one_mod_four
    Nat.1.mod(Nat.4) = Nat.1
    if Nat.3.congr_mod(Nat.1, Nat.4) {
        Nat.3.mod(Nat.4) = Nat.1.mod(Nat.4)
        Nat.3 = Nat.1
        three_ne_one
        false
    }
}

/// `3 ≢ 1 (mod 8)`.
theorem not_three_congr_one_mod_eight {
    not Nat.3.congr_mod(Nat.1, Nat.8)
} by {
    three_mod_eight
    Nat.3.mod(Nat.8) = Nat.3
    one_mod_eight
    Nat.1.mod(Nat.8) = Nat.1
    if Nat.3.congr_mod(Nat.1, Nat.8) {
        Nat.3.mod(Nat.8) = Nat.1.mod(Nat.8)
        Nat.3 = Nat.1
        three_ne_one
        false
    }
}

/// If some unit `a` is not congruent to `1` modulo `n`, then `1` is not a
/// universal exponent modulo `n`.
theorem not_carmichael_witness_one(n: Nat, a: Nat) {
    a.coprime(n) and not a.congr_mod(Nat.1, n)
        implies not carmichael_witness(n)(Nat.1)
} by {
    if a.coprime(n) and not a.congr_mod(Nat.1, n) {
        if carmichael_witness(n)(Nat.1) {
            carmichael_witness_member(n, Nat.1, a)
            a.pow(Nat.1).congr_mod(Nat.1, n)
            exp_one(a)
            a.pow(Nat.1) = a
            a.congr_mod(Nat.1, n)
            false
        }
    }
}

/// `λ(2) = 1`.
theorem carmichael_two {
    carmichael(Nat.2) = Nat.1
} by {
    // `1` is a universal exponent: the only unit modulo `2` is `1`.
    forall(a: Nat) {
        if a.coprime(Nat.2) {
            coprime_two_congr_one(a)
            a.congr_mod(Nat.1, Nat.2)
            exp_one(a)
            a.pow(Nat.1) = a
            a.pow(Nat.1).congr_mod(Nat.1, Nat.2)
        }
    }
    Nat.0 < Nat.1
    carmichael_witness_apply(Nat.2, Nat.1)
    carmichael_witness(Nat.2)(Nat.1) =
        (Nat.0 < Nat.1 and forall(a: Nat) {
            a.coprime(Nat.2) implies a.pow(Nat.1).congr_mod(Nat.1, Nat.2)
        })
    Nat.0 < Nat.1 and forall(a: Nat) {
        a.coprime(Nat.2) implies a.pow(Nat.1).congr_mod(Nat.1, Nat.2)
    }
    carmichael_witness(Nat.2)(Nat.1)
    // No positive `j < 1` exists.
    forall(j: Nat) {
        if Nat.0 < j and j < Nat.1 {
            lt_suc_right(j, Nat.0)
            j = Nat.0 or j < Nat.0
            if j < Nat.0 {
                not_lt_zero(j)
                false
            }
            j = Nat.0
            Nat.0 < j
            Nat.0 < Nat.0
            lt_not_ref(Nat.0)
            false
        }
    }
    is_carmichael_one(Nat.2)
    is_carmichael(Nat.2, Nat.1)
    carmichael_eq_of_is_carmichael(Nat.2, Nat.1)
    carmichael(Nat.2) = Nat.1
}

/// `λ(4) = 2`.
theorem carmichael_four {
    carmichael(Nat.4) = Nat.2
} by {
    // `2` is a universal exponent: every unit modulo `4` squares to `1`.
    forall(a: Nat) {
        if a.coprime(Nat.4) {
            unit_four_square_one(a)
            a.pow(Nat.2).congr_mod(Nat.1, Nat.4)
        }
    }
    Nat.0 < Nat.2
    carmichael_witness_apply(Nat.4, Nat.2)
    carmichael_witness(Nat.4)(Nat.2) =
        (Nat.0 < Nat.2 and forall(a: Nat) {
            a.coprime(Nat.4) implies a.pow(Nat.2).congr_mod(Nat.1, Nat.4)
        })
    Nat.0 < Nat.2 and forall(a: Nat) {
        a.coprime(Nat.4) implies a.pow(Nat.2).congr_mod(Nat.1, Nat.4)
    }
    carmichael_witness(Nat.4)(Nat.2)
    // `1` is not universal: the unit `3` is not congruent to `1` modulo `4`.
    forall(j: Nat) {
        if Nat.0 < j and j < Nat.2 {
            if j = Nat.1 {
                not_carmichael_witness_one(Nat.4, Nat.3)
                three_coprime_four
                not_three_congr_one_mod_four
                not carmichael_witness(Nat.4)(Nat.1)
                not carmichael_witness(Nat.4)(j)
            } else {
                j != Nat.1
                lt_suc_right(j, Nat.1)
                j < Nat.1
                lt_suc_right(j, Nat.0)
                j = Nat.0 or j < Nat.0
                if j < Nat.0 {
                    not_lt_zero(j)
                    false
                }
                j = Nat.0
                Nat.0 < j
                Nat.0 < Nat.0
                lt_not_ref(Nat.0)
                false
            }
        }
    }
    not carmichael_witness(Nat.4)(Nat.1)
    is_carmichael_two(Nat.4)
    is_carmichael(Nat.4, Nat.2)
    carmichael_eq_of_is_carmichael(Nat.4, Nat.2)
    carmichael(Nat.4) = Nat.2
}

/// `λ(8) = 2`.
theorem carmichael_eight {
    carmichael(Nat.8) = Nat.2
} by {
    // `2` is a universal exponent: every unit modulo `8` squares to `1`.
    forall(a: Nat) {
        if a.coprime(Nat.8) {
            unit_eight_square_one(a)
            a.pow(Nat.2).congr_mod(Nat.1, Nat.8)
        }
    }
    Nat.0 < Nat.2
    carmichael_witness_apply(Nat.8, Nat.2)
    carmichael_witness(Nat.8)(Nat.2) =
        (Nat.0 < Nat.2 and forall(a: Nat) {
            a.coprime(Nat.8) implies a.pow(Nat.2).congr_mod(Nat.1, Nat.8)
        })
    Nat.0 < Nat.2 and forall(a: Nat) {
        a.coprime(Nat.8) implies a.pow(Nat.2).congr_mod(Nat.1, Nat.8)
    }
    carmichael_witness(Nat.8)(Nat.2)
    // `1` is not universal: the unit `3` is not congruent to `1` modulo `8`.
    forall(j: Nat) {
        if Nat.0 < j and j < Nat.2 {
            if j = Nat.1 {
                not_carmichael_witness_one(Nat.8, Nat.3)
                three_coprime_eight
                not_three_congr_one_mod_eight
                not carmichael_witness(Nat.8)(Nat.1)
                not carmichael_witness(Nat.8)(j)
            } else {
                j != Nat.1
                lt_suc_right(j, Nat.1)
                j < Nat.1
                lt_suc_right(j, Nat.0)
                j = Nat.0 or j < Nat.0
                if j < Nat.0 {
                    not_lt_zero(j)
                    false
                }
                j = Nat.0
                Nat.0 < j
                Nat.0 < Nat.0
                lt_not_ref(Nat.0)
                false
            }
        }
    }
    not carmichael_witness(Nat.8)(Nat.1)
    is_carmichael_two(Nat.8)
    is_carmichael(Nat.8, Nat.2)
    carmichael_eq_of_is_carmichael(Nat.8, Nat.2)
    carmichael(Nat.8) = Nat.2
}

// ---------------------------------------------------------------------------
// Unconditional λ(p) = p - 1 for odd primes (not yet proved).
//
// The statement below is left commented out: the unconditional proof needs an
// element of multiplicative order `p - 1` modulo the prime `p` (a primitive
// root), i.e. `exists(g: Nat) { g.coprime(p) and multiplicative_order_mod(g, p) = p - Nat.1 }`.
// The library currently has no existence theorem for primitive roots modulo a
// prime; `carmichael_prime_of_generator` above proves `λ(p) = p - 1` under
// exactly that hypothesis. With a primitive root in hand, the remaining
// arguments are in place: `p - 1` is a universal exponent (Fermat), and
// `is_carmichael_prime` proves minimality.
//
// theorem carmichael_prime_unconditional(p: Nat) {
//     p.is_prime and p != Nat.2 implies carmichael(p) = p - Nat.1
// }
