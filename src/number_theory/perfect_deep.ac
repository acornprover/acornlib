// Perfect numbers, deepened: the third and fourth perfect numbers (496 and
// 8128), the abundant/deficient classification, and the shape of the
// Euclid-Euler theorem.
//
// The classical facts proved here are:
//
//   (a) 496 = 2^4 (2^5 - 1) is perfect: 2^5 - 1 = 31 is prime, so Euclid's
//       construction (mersenne_perfect.ac) applies, and 2^4 (2^5 - 1) = 496;
//   (b) 8128 = 2^6 (2^7 - 1) is perfect: 2^7 - 1 = 127 is prime (proved
//       here by a divisor check up to 11), and 2^6 (2^7 - 1) = 8128;
//   (c) twelve is abundant (sigma(12) = 28 > 24) and eight is deficient
//       (sigma(8) = 15 < 16);
//   (d) every multiple of a perfect number is abundant; the general statement
//       is recorded at the end, and the instance 2 * 6 = 12 is proved here;
//   (e) the Euclid-Euler theorem (the converse direction is future work) is
//       stated as a comment.
//
// Definitions of `is_abundant` and `is_deficient` live here; `is_perfect`
// is defined in perfect_numbers.ac.

from nat import Nat, divides_trans, divides_sub, add_imp_sub, add_comm,
    add_assoc, add_one_right, suc_sub_one, alt_suc_ne_zero, mul_comm,
    mul_one_left, mul_two_left, exp_add, exp_one, lte_mul_both, lte_trans,
    lt_suc, lt_suc_right, lt_or_lte, lt_imp_lte_suc, lte_and_lt, lt_not_ref,
    lte_imp_not_lt, gcd_of_prime, read_add_single, read_add_read,
    read_mul_single, read_read_carry, nat_mul_2_6, nat_mul_2_8
from number_theory.zsigmondy import lt_ne, divides_suc_pair_imp_one,
    not_divides_of_lt, lt_zero_two, lt_zero_one, two_pow_three, two_pow_four,
    three_ne_one
from number_theory.coprime import coprime_comm
from number_theory.divisor_sum import nat_sigma, nat_sigma_prime
from number_theory.perfect_numbers import is_perfect, nat_sigma_four
from number_theory.mersenne_perfect import euclid_construction,
    two_pow_five_sub_one_is_prime, two_pow_5_sub_one, nat_2_pow_5,
    nat_2_mul_32, nat_2_mul_64, two_is_prime, three_is_prime, two_ne_one,
    read_pos
from number_theory.sigma_multiplicative import nat_sigma_prime_pow_mult
from number_theory.liouville import nat_three_not_divides_four
from number_theory.dirichlet import nat_sigma_multiplicative
from number_theory.arithmetic_functions import is_multiplicative_nat_fn,
    multiplicative_nat_fn_apply
numerals Nat

// ---------------------------------------------------------------------------
// Abundant and deficient numbers.
// ---------------------------------------------------------------------------

/// A natural number `n` is abundant when the sum of its positive divisors
/// exceeds twice the number: `sigma(n) > 2 n`.
define is_abundant(n: Nat) -> Bool {
    Nat.2 * n < nat_sigma(n)
}

/// A natural number `n` is deficient when the sum of its positive divisors
/// falls short of twice the number: `sigma(n) < 2 n`.
define is_deficient(n: Nat) -> Bool {
    nat_sigma(n) < Nat.2 * n
}

// ---------------------------------------------------------------------------
// Decimal arithmetic, built explicitly with the base-10 read machinery so
// that every computation below is deterministic.
// ---------------------------------------------------------------------------

/// `7 + 5 = 12`, via `7 = 5 + 2` and `5 + 5 = 10`.
theorem nat_7_add_5 {
    Nat.7 + Nat.5 = Nat.12
} by {
    Nat.5 + Nat.2 = Nat.7
    Nat.7 + Nat.5 = (Nat.5 + Nat.2) + Nat.5
    add_assoc(Nat.5, Nat.2, Nat.5)
    (Nat.5 + Nat.2) + Nat.5 = Nat.5 + (Nat.2 + Nat.5)
    add_comm(Nat.2, Nat.5)
    Nat.2 + Nat.5 = Nat.5 + Nat.2
    Nat.5 + (Nat.2 + Nat.5) = Nat.5 + (Nat.5 + Nat.2)
    add_assoc(Nat.5, Nat.5, Nat.2)
    Nat.5 + (Nat.5 + Nat.2) = (Nat.5 + Nat.5) + Nat.2
    Nat.5 + Nat.5 = Nat.10
    (Nat.5 + Nat.5) + Nat.2 = Nat.10 + Nat.2
    Nat.10 + Nat.2 = Nat.12
    Nat.7 + Nat.5 = Nat.12
}

/// `8 + 4 = 12`, via `4 = 2 + 2` and `8 + 2 = 10`.
theorem nat_8_add_4 {
    Nat.8 + Nat.4 = Nat.12
} by {
    Nat.2 + Nat.2 = Nat.4
    Nat.8 + Nat.4 = Nat.8 + (Nat.2 + Nat.2)
    add_assoc(Nat.8, Nat.2, Nat.2)
    Nat.8 + (Nat.2 + Nat.2) = (Nat.8 + Nat.2) + Nat.2
    Nat.8 + Nat.2 = Nat.10
    (Nat.8 + Nat.2) + Nat.2 = Nat.10 + Nat.2
    Nat.10 + Nat.2 = Nat.12
    Nat.8 + Nat.4 = Nat.12
}

/// `42 + 2 = 44`.
theorem nat_42_add_2 {
    Nat.42 + Nat.2 = Nat.44
} by {
    Nat.42 = Nat.4.read(Nat.2)
    read_add_single(Nat.4, Nat.2, Nat.2)
    Nat.4.read(Nat.2) + Nat.2 = Nat.4.read(Nat.2 + Nat.2)
    Nat.2 + Nat.2 = Nat.4
    Nat.4.read(Nat.2 + Nat.2) = Nat.4.read(Nat.4)
    Nat.4.read(Nat.4) = Nat.44
    Nat.42 + Nat.2 = Nat.44
}

/// `76 + 4 = 80`.
theorem nat_76_add_4 {
    Nat.76 + Nat.4 = Nat.80
} by {
    Nat.76 = Nat.7.read(Nat.6)
    read_add_single(Nat.7, Nat.6, Nat.4)
    Nat.7.read(Nat.6) + Nat.4 = Nat.7.read(Nat.6 + Nat.4)
    Nat.6 + Nat.4 = Nat.10
    Nat.7.read(Nat.6 + Nat.4) = Nat.7.read(Nat.10)
    Nat.7.read(Nat.10) = Nat.7.read(Nat.10 * Nat.1 + Nat.0)
    read_read_carry(Nat.7, Nat.1, Nat.0)
    Nat.7.read(Nat.10 * Nat.1 + Nat.0) = (Nat.7 + Nat.1).read(Nat.0)
    Nat.7 + Nat.1 = Nat.8
    (Nat.7 + Nat.1).read(Nat.0) = Nat.8.read(Nat.0)
    Nat.8.read(Nat.0) = Nat.80
    Nat.76 + Nat.4 = Nat.80
}

/// `768 + 44 = 812`, by adding the digit pairs `76 + 4` and `8 + 4`.
theorem nat_768_add_44 {
    Nat.768 + Nat.44 = Nat.812
} by {
    Nat.768 = Nat.76.read(Nat.8)
    Nat.44 = Nat.4.read(Nat.4)
    read_add_read(Nat.76, Nat.8, Nat.4, Nat.4)
    Nat.76.read(Nat.8) + Nat.4.read(Nat.4) = (Nat.76 + Nat.4).read(Nat.8 + Nat.4)
    nat_76_add_4
    Nat.76 + Nat.4 = Nat.80
    nat_8_add_4
    Nat.8 + Nat.4 = Nat.12
    (Nat.76 + Nat.4).read(Nat.8 + Nat.4) = Nat.80.read(Nat.12)
    Nat.80.read(Nat.12) = Nat.80.read(Nat.10 * Nat.1 + Nat.2)
    read_read_carry(Nat.80, Nat.1, Nat.2)
    Nat.80.read(Nat.10 * Nat.1 + Nat.2) = (Nat.80 + Nat.1).read(Nat.2)
    Nat.80 + Nat.1 = Nat.81
    (Nat.80 + Nat.1).read(Nat.2) = Nat.81.read(Nat.2)
    Nat.81.read(Nat.2) = Nat.812
    Nat.768 + Nat.44 = Nat.812
}

/// `127 + 17 = 144`, by adding the digit pairs `12 + 1` and `7 + 7`.
theorem nat_127_add_17 {
    Nat.127 + Nat.17 = Nat.144
} by {
    Nat.127 = Nat.12.read(Nat.7)
    Nat.17 = Nat.1.read(Nat.7)
    read_add_read(Nat.12, Nat.7, Nat.1, Nat.7)
    Nat.12.read(Nat.7) + Nat.1.read(Nat.7) = (Nat.12 + Nat.1).read(Nat.7 + Nat.7)
    Nat.12 + Nat.1 = Nat.13
    Nat.7 + Nat.7 = Nat.14
    (Nat.12 + Nat.1).read(Nat.7 + Nat.7) = Nat.13.read(Nat.14)
    Nat.13.read(Nat.14) = Nat.13.read(Nat.10 * Nat.1 + Nat.4)
    read_read_carry(Nat.13, Nat.1, Nat.4)
    Nat.13.read(Nat.10 * Nat.1 + Nat.4) = (Nat.13 + Nat.1).read(Nat.4)
    Nat.13 + Nat.1 = Nat.14
    (Nat.13 + Nat.1).read(Nat.4) = Nat.14.read(Nat.4)
    Nat.14.read(Nat.4) = Nat.144
    Nat.127 + Nat.17 = Nat.144
}

/// `17 != 0`, since a positive leading digit makes a read number positive.
theorem nat_17_ne_zero {
    Nat.17 != Nat.0
} by {
    Nat.17 = Nat.1.read(Nat.7)
    read_pos(Nat.1, Nat.7)
    lt_zero_one
    Nat.0 < Nat.1
    Nat.0 < Nat.17
    Nat.17 != Nat.0
}

/// `2 * 63 = 126`.
theorem nat_2_mul_63 {
    Nat.2 * Nat.63 = Nat.126
} by {
    Nat.63 = Nat.6.read(Nat.3)
    read_mul_single(Nat.6, Nat.3, Nat.2)
    Nat.6.read(Nat.3) * Nat.2 = (Nat.6 * Nat.2).read(Nat.3 * Nat.2)
    Nat.6 * Nat.2 = Nat.12
    Nat.3 * Nat.2 = Nat.6
    Nat.6.read(Nat.3) * Nat.2 = Nat.12.read(Nat.6)
    Nat.12.read(Nat.6) = Nat.126
    Nat.63 * Nat.2 = Nat.126
    mul_comm(Nat.63, Nat.2)
    Nat.63 * Nat.2 = Nat.2 * Nat.63
    Nat.2 * Nat.63 = Nat.126
}

/// `3 * 42 = 126`.
theorem nat_3_mul_42 {
    Nat.3 * Nat.42 = Nat.126
} by {
    Nat.42 = Nat.4.read(Nat.2)
    read_mul_single(Nat.4, Nat.2, Nat.3)
    Nat.4.read(Nat.2) * Nat.3 = (Nat.4 * Nat.3).read(Nat.2 * Nat.3)
    Nat.4 * Nat.3 = Nat.12
    Nat.2 * Nat.3 = Nat.6
    Nat.4.read(Nat.2) * Nat.3 = Nat.12.read(Nat.6)
    Nat.12.read(Nat.6) = Nat.126
    Nat.42 * Nat.3 = Nat.126
    mul_comm(Nat.42, Nat.3)
    Nat.42 * Nat.3 = Nat.3 * Nat.42
    Nat.3 * Nat.42 = Nat.126
}

/// `5 * 25 = 125`.
theorem nat_5_mul_25 {
    Nat.5 * Nat.25 = Nat.125
} by {
    Nat.25 = Nat.2.read(Nat.5)
    read_mul_single(Nat.2, Nat.5, Nat.5)
    Nat.2.read(Nat.5) * Nat.5 = (Nat.2 * Nat.5).read(Nat.5 * Nat.5)
    Nat.2 * Nat.5 = Nat.10
    Nat.5 * Nat.5 = Nat.25
    Nat.2.read(Nat.5) * Nat.5 = Nat.10.read(Nat.25)
    Nat.10.read(Nat.25) = Nat.10.read(Nat.10 * Nat.2 + Nat.5)
    read_read_carry(Nat.10, Nat.2, Nat.5)
    Nat.10.read(Nat.10 * Nat.2 + Nat.5) = (Nat.10 + Nat.2).read(Nat.5)
    Nat.10 + Nat.2 = Nat.12
    (Nat.10 + Nat.2).read(Nat.5) = Nat.12.read(Nat.5)
    Nat.12.read(Nat.5) = Nat.125
    Nat.25 * Nat.5 = Nat.125
    mul_comm(Nat.25, Nat.5)
    Nat.25 * Nat.5 = Nat.5 * Nat.25
    Nat.5 * Nat.25 = Nat.125
}

/// `127 - 125 = 2`, via `125 + 2 = 127`.
theorem nat_127_sub_125 {
    Nat.127 - Nat.125 = Nat.2
} by {
    Nat.125 = Nat.12.read(Nat.5)
    read_add_single(Nat.12, Nat.5, Nat.2)
    Nat.12.read(Nat.5) + Nat.2 = Nat.12.read(Nat.5 + Nat.2)
    Nat.5 + Nat.2 = Nat.7
    Nat.12.read(Nat.5 + Nat.2) = Nat.12.read(Nat.7)
    Nat.12.read(Nat.7) = Nat.127
    Nat.125 + Nat.2 = Nat.127
    add_comm(Nat.125, Nat.2)
    Nat.125 + Nat.2 = Nat.2 + Nat.125
    Nat.2 + Nat.125 = Nat.127
    add_imp_sub(Nat.2, Nat.125, Nat.127)
    Nat.127 - Nat.125 = Nat.2
}

/// `7 * 18 = 126`.
theorem nat_7_mul_18 {
    Nat.7 * Nat.18 = Nat.126
} by {
    Nat.18 = Nat.1.read(Nat.8)
    read_mul_single(Nat.1, Nat.8, Nat.7)
    Nat.1.read(Nat.8) * Nat.7 = (Nat.1 * Nat.7).read(Nat.8 * Nat.7)
    Nat.1 * Nat.7 = Nat.7
    Nat.8 * Nat.7 = Nat.56
    Nat.1.read(Nat.8) * Nat.7 = Nat.7.read(Nat.56)
    Nat.7.read(Nat.56) = Nat.7.read(Nat.10 * Nat.5 + Nat.6)
    read_read_carry(Nat.7, Nat.5, Nat.6)
    Nat.7.read(Nat.10 * Nat.5 + Nat.6) = (Nat.7 + Nat.5).read(Nat.6)
    nat_7_add_5
    Nat.7 + Nat.5 = Nat.12
    (Nat.7 + Nat.5).read(Nat.6) = Nat.12.read(Nat.6)
    Nat.12.read(Nat.6) = Nat.126
    Nat.18 * Nat.7 = Nat.126
    mul_comm(Nat.18, Nat.7)
    Nat.18 * Nat.7 = Nat.7 * Nat.18
    Nat.7 * Nat.18 = Nat.126
}

/// `11 * 11 = 121`.
theorem nat_11_mul_11 {
    Nat.11 * Nat.11 = Nat.121
} by {
    Nat.11 = Nat.1.read(Nat.1)
    read_mul_single(Nat.1, Nat.1, Nat.11)
    Nat.1.read(Nat.1) * Nat.11 = (Nat.1 * Nat.11).read(Nat.1 * Nat.11)
    Nat.1 * Nat.11 = Nat.11
    Nat.1.read(Nat.1) * Nat.11 = Nat.11.read(Nat.11)
    Nat.11.read(Nat.11) = Nat.11.read(Nat.10 * Nat.1 + Nat.1)
    read_read_carry(Nat.11, Nat.1, Nat.1)
    Nat.11.read(Nat.10 * Nat.1 + Nat.1) = (Nat.11 + Nat.1).read(Nat.1)
    Nat.11 + Nat.1 = Nat.12
    (Nat.11 + Nat.1).read(Nat.1) = Nat.12.read(Nat.1)
    Nat.12.read(Nat.1) = Nat.121
    Nat.11 * Nat.11 = Nat.121
}

/// `3 * 16 = 48`.
theorem nat_3_mul_16 {
    Nat.3 * Nat.16 = Nat.48
} by {
    Nat.16 = Nat.1.read(Nat.6)
    read_mul_single(Nat.1, Nat.6, Nat.3)
    Nat.1.read(Nat.6) * Nat.3 = (Nat.1 * Nat.3).read(Nat.6 * Nat.3)
    Nat.1 * Nat.3 = Nat.3
    Nat.6 * Nat.3 = Nat.18
    Nat.1.read(Nat.6) * Nat.3 = Nat.3.read(Nat.18)
    Nat.3.read(Nat.18) = Nat.3.read(Nat.10 * Nat.1 + Nat.8)
    read_read_carry(Nat.3, Nat.1, Nat.8)
    Nat.3.read(Nat.10 * Nat.1 + Nat.8) = (Nat.3 + Nat.1).read(Nat.8)
    Nat.3 + Nat.1 = Nat.4
    (Nat.3 + Nat.1).read(Nat.8) = Nat.4.read(Nat.8)
    Nat.4.read(Nat.8) = Nat.48
    Nat.3 * Nat.16 = Nat.48
}

/// `16 * 31 = 496`.
theorem nat_16_mul_31 {
    Nat.16 * Nat.31 = Nat.496
} by {
    Nat.31 = Nat.3.read(Nat.1)
    read_mul_single(Nat.3, Nat.1, Nat.16)
    Nat.3.read(Nat.1) * Nat.16 = (Nat.3 * Nat.16).read(Nat.1 * Nat.16)
    nat_3_mul_16
    Nat.3 * Nat.16 = Nat.48
    Nat.1 * Nat.16 = Nat.16
    Nat.3.read(Nat.1) * Nat.16 = Nat.48.read(Nat.16)
    Nat.48.read(Nat.16) = Nat.48.read(Nat.10 * Nat.1 + Nat.6)
    read_read_carry(Nat.48, Nat.1, Nat.6)
    Nat.48.read(Nat.10 * Nat.1 + Nat.6) = (Nat.48 + Nat.1).read(Nat.6)
    Nat.48 + Nat.1 = Nat.49
    (Nat.48 + Nat.1).read(Nat.6) = Nat.49.read(Nat.6)
    Nat.49.read(Nat.6) = Nat.496
    Nat.31 * Nat.16 = Nat.496
    mul_comm(Nat.31, Nat.16)
    Nat.31 * Nat.16 = Nat.16 * Nat.31
    Nat.16 * Nat.31 = Nat.496
}

/// `7 * 64 = 448`.
theorem nat_7_mul_64 {
    Nat.7 * Nat.64 = Nat.448
} by {
    Nat.64 = Nat.6.read(Nat.4)
    read_mul_single(Nat.6, Nat.4, Nat.7)
    Nat.6.read(Nat.4) * Nat.7 = (Nat.6 * Nat.7).read(Nat.4 * Nat.7)
    Nat.6 * Nat.7 = Nat.42
    Nat.4 * Nat.7 = Nat.28
    Nat.6.read(Nat.4) * Nat.7 = Nat.42.read(Nat.28)
    Nat.42.read(Nat.28) = Nat.42.read(Nat.10 * Nat.2 + Nat.8)
    read_read_carry(Nat.42, Nat.2, Nat.8)
    Nat.42.read(Nat.10 * Nat.2 + Nat.8) = (Nat.42 + Nat.2).read(Nat.8)
    nat_42_add_2
    Nat.42 + Nat.2 = Nat.44
    (Nat.42 + Nat.2).read(Nat.8) = Nat.44.read(Nat.8)
    Nat.44.read(Nat.8) = Nat.448
    Nat.64 * Nat.7 = Nat.448
    mul_comm(Nat.64, Nat.7)
    Nat.64 * Nat.7 = Nat.7 * Nat.64
    Nat.7 * Nat.64 = Nat.448
}

/// `12 * 64 = 768`.
theorem nat_12_mul_64 {
    Nat.12 * Nat.64 = Nat.768
} by {
    Nat.12 = Nat.1.read(Nat.2)
    read_mul_single(Nat.1, Nat.2, Nat.64)
    Nat.1.read(Nat.2) * Nat.64 = (Nat.1 * Nat.64).read(Nat.2 * Nat.64)
    Nat.1 * Nat.64 = Nat.64
    nat_2_mul_64
    Nat.2 * Nat.64 = Nat.128
    Nat.1.read(Nat.2) * Nat.64 = Nat.64.read(Nat.128)
    Nat.64.read(Nat.128) = Nat.64.read(Nat.10 * Nat.12 + Nat.8)
    read_read_carry(Nat.64, Nat.12, Nat.8)
    Nat.64.read(Nat.10 * Nat.12 + Nat.8) = (Nat.64 + Nat.12).read(Nat.8)
    Nat.64 + Nat.12 = Nat.76
    (Nat.64 + Nat.12).read(Nat.8) = Nat.76.read(Nat.8)
    Nat.76.read(Nat.8) = Nat.768
    Nat.12 * Nat.64 = Nat.768
}

/// `64 * 127 = 8128`.
theorem nat_64_mul_127 {
    Nat.64 * Nat.127 = Nat.8128
} by {
    Nat.127 = Nat.12.read(Nat.7)
    read_mul_single(Nat.12, Nat.7, Nat.64)
    Nat.12.read(Nat.7) * Nat.64 = (Nat.12 * Nat.64).read(Nat.7 * Nat.64)
    nat_12_mul_64
    Nat.12 * Nat.64 = Nat.768
    nat_7_mul_64
    Nat.7 * Nat.64 = Nat.448
    Nat.12.read(Nat.7) * Nat.64 = Nat.768.read(Nat.448)
    Nat.768.read(Nat.448) = Nat.768.read(Nat.10 * Nat.44 + Nat.8)
    read_read_carry(Nat.768, Nat.44, Nat.8)
    Nat.768.read(Nat.10 * Nat.44 + Nat.8) = (Nat.768 + Nat.44).read(Nat.8)
    nat_768_add_44
    Nat.768 + Nat.44 = Nat.812
    (Nat.768 + Nat.44).read(Nat.8) = Nat.812.read(Nat.8)
    Nat.812.read(Nat.8) = Nat.8128
    Nat.127 * Nat.64 = Nat.8128
    mul_comm(Nat.127, Nat.64)
    Nat.127 * Nat.64 = Nat.64 * Nat.127
    Nat.64 * Nat.127 = Nat.8128
}

/// `12 * 12 = 144`.
theorem nat_12_mul_12 {
    Nat.12 * Nat.12 = Nat.144
} by {
    Nat.12 = Nat.1.read(Nat.2)
    read_mul_single(Nat.1, Nat.2, Nat.12)
    Nat.1.read(Nat.2) * Nat.12 = (Nat.1 * Nat.12).read(Nat.2 * Nat.12)
    Nat.1 * Nat.12 = Nat.12
    Nat.2 * Nat.12 = Nat.24
    Nat.1.read(Nat.2) * Nat.12 = Nat.12.read(Nat.24)
    Nat.12.read(Nat.24) = Nat.12.read(Nat.10 * Nat.2 + Nat.4)
    read_read_carry(Nat.12, Nat.2, Nat.4)
    Nat.12.read(Nat.10 * Nat.2 + Nat.4) = (Nat.12 + Nat.2).read(Nat.4)
    Nat.12 = Nat.1.read(Nat.2)
    read_add_single(Nat.1, Nat.2, Nat.2)
    Nat.1.read(Nat.2) + Nat.2 = Nat.1.read(Nat.2 + Nat.2)
    Nat.2 + Nat.2 = Nat.4
    Nat.1.read(Nat.2 + Nat.2) = Nat.1.read(Nat.4)
    Nat.1.read(Nat.4) = Nat.14
    Nat.12 + Nat.2 = Nat.14
    (Nat.12 + Nat.2).read(Nat.4) = Nat.14.read(Nat.4)
    Nat.14.read(Nat.4) = Nat.144
    Nat.12 * Nat.12 = Nat.144
}

// ---------------------------------------------------------------------------
// Powers of two, up to 2^7.
// ---------------------------------------------------------------------------

/// `2^5 = 32` (from mersenne_perfect.ac).
theorem nat_two_pow_five {
    Nat.2.pow(Nat.5) = Nat.32
} by {
    nat_2_pow_5
    Nat.2.pow(Nat.5) = Nat.32
}

/// `2^6 = 64`.
theorem nat_2_pow_6 {
    Nat.2.pow(Nat.6) = Nat.64
} by {
    exp_add(Nat.2, Nat.5, Nat.1)
    Nat.2.pow(Nat.5 + Nat.1) = Nat.2.pow(Nat.5) * Nat.2.pow(Nat.1)
    Nat.5 + Nat.1 = Nat.6
    Nat.2.pow(Nat.6) = Nat.2.pow(Nat.5) * Nat.2.pow(Nat.1)
    exp_one(Nat.2)
    Nat.2.pow(Nat.1) = Nat.2
    Nat.2.pow(Nat.6) = Nat.2.pow(Nat.5) * Nat.2
    nat_two_pow_five
    Nat.2.pow(Nat.5) = Nat.32
    Nat.2.pow(Nat.6) = Nat.32 * Nat.2
    mul_comm(Nat.32, Nat.2)
    Nat.32 * Nat.2 = Nat.2 * Nat.32
    nat_2_mul_32
    Nat.2 * Nat.32 = Nat.64
    Nat.32 * Nat.2 = Nat.64
    Nat.2.pow(Nat.6) = Nat.64
}

/// `2^7 = 128`.
theorem nat_2_pow_7 {
    Nat.2.pow(Nat.7) = Nat.128
} by {
    exp_add(Nat.2, Nat.6, Nat.1)
    Nat.2.pow(Nat.6 + Nat.1) = Nat.2.pow(Nat.6) * Nat.2.pow(Nat.1)
    Nat.6 + Nat.1 = Nat.7
    Nat.2.pow(Nat.7) = Nat.2.pow(Nat.6) * Nat.2.pow(Nat.1)
    exp_one(Nat.2)
    Nat.2.pow(Nat.1) = Nat.2
    Nat.2.pow(Nat.7) = Nat.2.pow(Nat.6) * Nat.2
    nat_2_pow_6
    Nat.2.pow(Nat.6) = Nat.64
    Nat.2.pow(Nat.7) = Nat.64 * Nat.2
    mul_comm(Nat.64, Nat.2)
    Nat.64 * Nat.2 = Nat.2 * Nat.64
    nat_2_mul_64
    Nat.2 * Nat.64 = Nat.128
    Nat.64 * Nat.2 = Nat.128
    Nat.2.pow(Nat.7) = Nat.128
}

/// `2^7 - 1 = 127`.
theorem nat_2_pow_7_sub_one {
    Nat.2.pow(Nat.7) - Nat.1 = Nat.127
} by {
    nat_2_pow_7
    Nat.2.pow(Nat.7) = Nat.128
    Nat.127 + Nat.1 = Nat.128
    add_imp_sub(Nat.127, Nat.1, Nat.128)
    Nat.128 - Nat.1 = Nat.127
    Nat.2.pow(Nat.7) - Nat.1 = Nat.127
}

// ---------------------------------------------------------------------------
// 127 is prime: no divisor strictly between 1 and 127 divides it.  A proper
// factor pair `b * c = 127` would force one of `b, c` to lie in
// `{2, ..., 11}` (otherwise `b * c >= 12 * 12 = 144 > 127`), and each of
// `2, ..., 11` is ruled out individually.
// ---------------------------------------------------------------------------

/// `1 < 127`.
theorem nat_1_lt_127 {
    Nat.1 < Nat.127
} by {
    Nat.1 + Nat.126 = Nat.127
    exists(c: Nat) { Nat.1 + c = Nat.127 }
    Nat.1 <= Nat.127
    lt_ne(Nat.1, Nat.127, Nat.126)
    Nat.1 + Nat.126 = Nat.127
    alt_suc_ne_zero(Nat.125)
    Nat.125.suc != Nat.0
    Nat.125.suc = Nat.126
    Nat.126 != Nat.0
    Nat.1 != Nat.127
    Nat.1 < Nat.127
}

/// `2 < 5`.
theorem nat_2_lt_5 {
    Nat.2 < Nat.5
} by {
    Nat.2 + Nat.3 = Nat.5
    exists(c: Nat) { Nat.2 + c = Nat.5 }
    Nat.2 <= Nat.5
    lt_ne(Nat.2, Nat.5, Nat.3)
    Nat.2 + Nat.3 = Nat.5
    Nat.3 != Nat.0
    Nat.2 != Nat.5
    Nat.2 < Nat.5
}

/// `0 < 6`.
theorem nat_0_lt_6 {
    Nat.0 < Nat.6
} by {
    Nat.0 + Nat.6 = Nat.6
    exists(c: Nat) { Nat.0 + c = Nat.6 }
    Nat.0 <= Nat.6
    lt_ne(Nat.0, Nat.6, Nat.6)
    Nat.0 + Nat.6 = Nat.6
    Nat.6 != Nat.0
    Nat.0 != Nat.6
    Nat.0 < Nat.6
}

/// `6 < 11`.
theorem nat_6_lt_11 {
    Nat.6 < Nat.11
} by {
    Nat.4 + Nat.1 = Nat.5
    Nat.6 + Nat.5 = Nat.6 + (Nat.4 + Nat.1)
    add_assoc(Nat.6, Nat.4, Nat.1)
    Nat.6 + (Nat.4 + Nat.1) = (Nat.6 + Nat.4) + Nat.1
    Nat.6 + Nat.4 = Nat.10
    (Nat.6 + Nat.4) + Nat.1 = Nat.10 + Nat.1
    add_one_right(Nat.10)
    Nat.10 + Nat.1 = Nat.10.suc
    Nat.10.suc = Nat.11
    Nat.10 + Nat.1 = Nat.11
    Nat.6 + Nat.5 = Nat.11
    exists(c: Nat) { Nat.6 + c = Nat.11 }
    Nat.6 <= Nat.11
    lt_ne(Nat.6, Nat.11, Nat.5)
    Nat.6 + Nat.5 = Nat.11
    alt_suc_ne_zero(Nat.4)
    Nat.4.suc != Nat.0
    Nat.4.suc = Nat.5
    Nat.5 != Nat.0
    Nat.6 != Nat.11
    Nat.6 < Nat.11
}

/// `7 != 1`.
theorem nat_7_ne_1 {
    Nat.7 != Nat.1
} by {
    lt_ne(Nat.1, Nat.7, Nat.6)
    Nat.1 + Nat.6 = Nat.7
    Nat.6 != Nat.0
    Nat.1 != Nat.7
    Nat.7 != Nat.1
}

/// Two does not divide one hundred twenty-seven.
theorem not_two_divides_127 {
    not Nat.2.divides(Nat.127)
} by {
    if Nat.2.divides(Nat.127) {
        nat_2_mul_63
        Nat.2 * Nat.63 = Nat.126
        exists(c: Nat) { Nat.2 * c = Nat.126 }
        Nat.2.divides(Nat.126)
        Nat.126 + Nat.1 = Nat.127
        divides_suc_pair_imp_one(Nat.2, Nat.126)
        Nat.2 = Nat.1
        two_ne_one
        false
    }
}

/// Three does not divide one hundred twenty-seven.
theorem not_three_divides_127 {
    not Nat.3.divides(Nat.127)
} by {
    if Nat.3.divides(Nat.127) {
        nat_3_mul_42
        Nat.3 * Nat.42 = Nat.126
        exists(c: Nat) { Nat.3 * c = Nat.126 }
        Nat.3.divides(Nat.126)
        Nat.126 + Nat.1 = Nat.127
        divides_suc_pair_imp_one(Nat.3, Nat.126)
        Nat.3 = Nat.1
        three_ne_one
        false
    }
}

/// Four does not divide one hundred twenty-seven, since two does not.
theorem not_four_divides_127 {
    not Nat.4.divides(Nat.127)
} by {
    if Nat.4.divides(Nat.127) {
        Nat.2 * Nat.2 = Nat.4
        exists(c: Nat) { Nat.2 * c = Nat.4 }
        Nat.2.divides(Nat.4)
        divides_trans(Nat.2, Nat.4, Nat.127)
        Nat.2.divides(Nat.4) and Nat.4.divides(Nat.127) implies Nat.2.divides(Nat.127)
        Nat.2.divides(Nat.127)
        not_two_divides_127
        false
    }
}

/// Five does not divide one hundred twenty-seven.
theorem not_five_divides_127 {
    not Nat.5.divides(Nat.127)
} by {
    if Nat.5.divides(Nat.127) {
        nat_5_mul_25
        Nat.5 * Nat.25 = Nat.125
        exists(c: Nat) { Nat.5 * c = Nat.125 }
        Nat.5.divides(Nat.125)
        divides_sub(Nat.127, Nat.125, Nat.5)
        Nat.5.divides(Nat.127) and Nat.5.divides(Nat.125) implies Nat.5.divides(Nat.127 - Nat.125)
        Nat.5.divides(Nat.127 - Nat.125)
        nat_127_sub_125
        Nat.127 - Nat.125 = Nat.2
        Nat.5.divides(Nat.2)
        not_divides_of_lt(Nat.5, Nat.2)
        Nat.0 < Nat.2 and Nat.2 < Nat.5 implies not Nat.5.divides(Nat.2)
        lt_zero_two
        Nat.0 < Nat.2
        nat_2_lt_5
        Nat.2 < Nat.5
        false
    }
}

/// Six does not divide one hundred twenty-seven, since two does not.
theorem not_six_divides_127 {
    not Nat.6.divides(Nat.127)
} by {
    if Nat.6.divides(Nat.127) {
        Nat.2 * Nat.3 = Nat.6
        exists(c: Nat) { Nat.2 * c = Nat.6 }
        Nat.2.divides(Nat.6)
        divides_trans(Nat.2, Nat.6, Nat.127)
        Nat.2.divides(Nat.6) and Nat.6.divides(Nat.127) implies Nat.2.divides(Nat.127)
        Nat.2.divides(Nat.127)
        not_two_divides_127
        false
    }
}

/// Seven does not divide one hundred twenty-seven.
theorem not_seven_divides_127 {
    not Nat.7.divides(Nat.127)
} by {
    if Nat.7.divides(Nat.127) {
        nat_7_mul_18
        Nat.7 * Nat.18 = Nat.126
        exists(c: Nat) { Nat.7 * c = Nat.126 }
        Nat.7.divides(Nat.126)
        Nat.126 + Nat.1 = Nat.127
        divides_suc_pair_imp_one(Nat.7, Nat.126)
        Nat.7 = Nat.1
        nat_7_ne_1
        false
    }
}

/// Eight does not divide one hundred twenty-seven, since two does not.
theorem not_eight_divides_127 {
    not Nat.8.divides(Nat.127)
} by {
    if Nat.8.divides(Nat.127) {
        Nat.2 * Nat.4 = Nat.8
        exists(c: Nat) { Nat.2 * c = Nat.8 }
        Nat.2.divides(Nat.8)
        divides_trans(Nat.2, Nat.8, Nat.127)
        Nat.2.divides(Nat.8) and Nat.8.divides(Nat.127) implies Nat.2.divides(Nat.127)
        Nat.2.divides(Nat.127)
        not_two_divides_127
        false
    }
}

/// Nine does not divide one hundred twenty-seven, since three does not.
theorem not_nine_divides_127 {
    not Nat.9.divides(Nat.127)
} by {
    if Nat.9.divides(Nat.127) {
        Nat.3 * Nat.3 = Nat.9
        exists(c: Nat) { Nat.3 * c = Nat.9 }
        Nat.3.divides(Nat.9)
        divides_trans(Nat.3, Nat.9, Nat.127)
        Nat.3.divides(Nat.9) and Nat.9.divides(Nat.127) implies Nat.3.divides(Nat.127)
        Nat.3.divides(Nat.127)
        not_three_divides_127
        false
    }
}

/// Ten does not divide one hundred twenty-seven, since two does not.
theorem not_ten_divides_127 {
    not Nat.10.divides(Nat.127)
} by {
    if Nat.10.divides(Nat.127) {
        Nat.2 * Nat.5 = Nat.10
        exists(c: Nat) { Nat.2 * c = Nat.10 }
        Nat.2.divides(Nat.10)
        divides_trans(Nat.2, Nat.10, Nat.127)
        Nat.2.divides(Nat.10) and Nat.10.divides(Nat.127) implies Nat.2.divides(Nat.127)
        Nat.2.divides(Nat.127)
        not_two_divides_127
        false
    }
}

/// `127 - 121 = 6`.
theorem nat_127_sub_121 {
    Nat.127 - Nat.121 = Nat.6
} by {
    Nat.6 + Nat.121 = Nat.127
    add_imp_sub(Nat.6, Nat.121, Nat.127)
    Nat.127 - Nat.121 = Nat.6
}

/// Eleven does not divide one hundred twenty-seven.
theorem not_eleven_divides_127 {
    not Nat.11.divides(Nat.127)
} by {
    if Nat.11.divides(Nat.127) {
        nat_11_mul_11
        Nat.11 * Nat.11 = Nat.121
        exists(c: Nat) { Nat.11 * c = Nat.121 }
        Nat.11.divides(Nat.121)
        divides_sub(Nat.127, Nat.121, Nat.11)
        Nat.11.divides(Nat.127) and Nat.11.divides(Nat.121) implies Nat.11.divides(Nat.127 - Nat.121)
        Nat.11.divides(Nat.127 - Nat.121)
        nat_127_sub_121
        Nat.127 - Nat.121 = Nat.6
        Nat.11.divides(Nat.6)
        not_divides_of_lt(Nat.11, Nat.6)
        Nat.0 < Nat.6 and Nat.6 < Nat.11 implies not Nat.11.divides(Nat.6)
        nat_0_lt_6
        Nat.0 < Nat.6
        nat_6_lt_11
        Nat.6 < Nat.11
        false
    }
}

/// No factorization `127 = b * c` with both factors greater than one exists:
/// one of the factors is at most `11` (otherwise `b * c >= 12 * 12 = 144`),
/// and each candidate factor `2, ..., 11` is ruled out individually.
theorem one_hundred_twenty_seven_composite_contradiction(b: Nat, c: Nat) {
    Nat.1 < b and Nat.1 < c implies not (Nat.127 = b * c)
} by {
    if Nat.1 < b and Nat.1 < c {
        if Nat.127 = b * c {
            Nat.127 = b * c
            exists(x: Nat) { b * x = Nat.127 }
            b.divides(Nat.127)
            if b <= Nat.11 {
                lt_suc(Nat.11)
                Nat.11 < Nat.11.suc
                Nat.11.suc = Nat.12
                Nat.11 < Nat.12
                lte_and_lt(b, Nat.11, Nat.12)
                b < Nat.12
                lt_suc_right(b, Nat.11)
                if b = Nat.11 {
                    Nat.11.divides(Nat.127)
                    not_eleven_divides_127
                    false
                } else {
                    b < Nat.11
                    lt_suc_right(b, Nat.10)
                    if b = Nat.10 {
                        Nat.10.divides(Nat.127)
                        not_ten_divides_127
                        false
                    } else {
                        b < Nat.10
                        lt_suc_right(b, Nat.9)
                        if b = Nat.9 {
                            Nat.9.divides(Nat.127)
                            not_nine_divides_127
                            false
                        } else {
                            b < Nat.9
                            lt_suc_right(b, Nat.8)
                            if b = Nat.8 {
                                Nat.8.divides(Nat.127)
                                not_eight_divides_127
                                false
                            } else {
                                b < Nat.8
                                lt_suc_right(b, Nat.7)
                                if b = Nat.7 {
                                    Nat.7.divides(Nat.127)
                                    not_seven_divides_127
                                    false
                                } else {
                                    b < Nat.7
                                    lt_suc_right(b, Nat.6)
                                    if b = Nat.6 {
                                        Nat.6.divides(Nat.127)
                                        not_six_divides_127
                                        false
                                    } else {
                                        b < Nat.6
                                        lt_suc_right(b, Nat.5)
                                        if b = Nat.5 {
                                            Nat.5.divides(Nat.127)
                                            not_five_divides_127
                                            false
                                        } else {
                                            b < Nat.5
                                            lt_suc_right(b, Nat.4)
                                            if b = Nat.4 {
                                                Nat.4.divides(Nat.127)
                                                not_four_divides_127
                                                false
                                            } else {
                                                b < Nat.4
                                                lt_suc_right(b, Nat.3)
                                                if b = Nat.3 {
                                                    Nat.3.divides(Nat.127)
                                                    not_three_divides_127
                                                    false
                                                } else {
                                                    b < Nat.3
                                                    lt_suc_right(b, Nat.2)
                                                    if b = Nat.2 {
                                                        Nat.2.divides(Nat.127)
                                                        not_two_divides_127
                                                        false
                                                    } else {
                                                        b < Nat.2
                                                        lt_imp_lte_suc(Nat.1, b)
                                                        Nat.2 <= b
                                                        lte_and_lt(Nat.2, b, Nat.2)
                                                        Nat.2 < Nat.2
                                                        lt_not_ref(Nat.2)
                                                        false
                                                    }
                                                }
                                            }
                                        }
                                    }
                                }
                            }
                        }
                    }
                }
            } else {
                not (b <= Nat.11)
                lt_or_lte(Nat.11, b)
                Nat.11 < b or b <= Nat.11
                Nat.11 < b
                lt_imp_lte_suc(Nat.11, b)
                Nat.11.suc <= b
                Nat.11.suc = Nat.12
                Nat.12 <= b
                if c <= Nat.11 {
                    Nat.127 = b * c
                    exists(x: Nat) { c * x = Nat.127 }
                    c.divides(Nat.127)
                    lt_suc(Nat.11)
                    Nat.11 < Nat.11.suc
                    Nat.11.suc = Nat.12
                    Nat.11 < Nat.12
                    lte_and_lt(c, Nat.11, Nat.12)
                    c < Nat.12
                    lt_suc_right(c, Nat.11)
                    if c = Nat.11 {
                        Nat.11.divides(Nat.127)
                        not_eleven_divides_127
                        false
                    } else {
                        c < Nat.11
                        lt_suc_right(c, Nat.10)
                        if c = Nat.10 {
                            Nat.10.divides(Nat.127)
                            not_ten_divides_127
                            false
                        } else {
                            c < Nat.10
                            lt_suc_right(c, Nat.9)
                            if c = Nat.9 {
                                Nat.9.divides(Nat.127)
                                not_nine_divides_127
                                false
                            } else {
                                c < Nat.9
                                lt_suc_right(c, Nat.8)
                                if c = Nat.8 {
                                    Nat.8.divides(Nat.127)
                                    not_eight_divides_127
                                    false
                                } else {
                                    c < Nat.8
                                    lt_suc_right(c, Nat.7)
                                    if c = Nat.7 {
                                        Nat.7.divides(Nat.127)
                                        not_seven_divides_127
                                        false
                                    } else {
                                        c < Nat.7
                                        lt_suc_right(c, Nat.6)
                                        if c = Nat.6 {
                                            Nat.6.divides(Nat.127)
                                            not_six_divides_127
                                            false
                                        } else {
                                            c < Nat.6
                                            lt_suc_right(c, Nat.5)
                                            if c = Nat.5 {
                                                Nat.5.divides(Nat.127)
                                                not_five_divides_127
                                                false
                                            } else {
                                                c < Nat.5
                                                lt_suc_right(c, Nat.4)
                                                if c = Nat.4 {
                                                    Nat.4.divides(Nat.127)
                                                    not_four_divides_127
                                                    false
                                                } else {
                                                    c < Nat.4
                                                    lt_suc_right(c, Nat.3)
                                                    if c = Nat.3 {
                                                        Nat.3.divides(Nat.127)
                                                        not_three_divides_127
                                                        false
                                                    } else {
                                                        c < Nat.3
                                                        lt_suc_right(c, Nat.2)
                                                        if c = Nat.2 {
                                                            Nat.2.divides(Nat.127)
                                                            not_two_divides_127
                                                            false
                                                        } else {
                                                            c < Nat.2
                                                            lt_imp_lte_suc(Nat.1, c)
                                                            Nat.2 <= c
                                                            lte_and_lt(Nat.2, c, Nat.2)
                                                            Nat.2 < Nat.2
                                                            lt_not_ref(Nat.2)
                                                            false
                                                        }
                                                    }
                                                }
                                            }
                                        }
                                    }
                                }
                            }
                        }
                    }
                } else {
                    not (c <= Nat.11)
                    lt_or_lte(Nat.11, c)
                    Nat.11 < c or c <= Nat.11
                    Nat.11 < c
                    lt_imp_lte_suc(Nat.11, c)
                    Nat.11.suc <= c
                    Nat.11.suc = Nat.12
                    Nat.12 <= c
                    lte_mul_both(Nat.12, Nat.12, c)
                    Nat.12 <= c implies Nat.12 * Nat.12 <= Nat.12 * c
                    Nat.12 * Nat.12 <= Nat.12 * c
                    nat_12_mul_12
                    Nat.12 * Nat.12 = Nat.144
                    Nat.144 <= Nat.12 * c
                    lte_mul_both(c, Nat.12, b)
                    Nat.12 <= b implies c * Nat.12 <= c * b
                    c * Nat.12 <= c * b
                    mul_comm(c, Nat.12)
                    c * Nat.12 = Nat.12 * c
                    mul_comm(c, b)
                    c * b = b * c
                    Nat.12 * c <= b * c
                    lte_trans(Nat.144, Nat.12 * c, b * c)
                    Nat.144 <= b * c
                    Nat.127 = b * c
                    Nat.144 <= Nat.127
                    nat_127_add_17
                    Nat.127 + Nat.17 = Nat.144
                    exists(d: Nat) { Nat.127 + d = Nat.144 }
                    Nat.127 <= Nat.144
                    lt_ne(Nat.127, Nat.144, Nat.17)
                    Nat.127 + Nat.17 = Nat.144
                    nat_17_ne_zero
                    Nat.17 != Nat.0
                    Nat.127 != Nat.144
                    Nat.127 < Nat.144
                    lte_imp_not_lt(Nat.144, Nat.127)
                    not (Nat.127 < Nat.144)
                    false
                }
            }
        }
        not (Nat.127 = b * c)
    }
}

/// One hundred twenty-seven is prime.
theorem one_hundred_twenty_seven_is_prime {
    Nat.127.is_prime
} by {
    nat_1_lt_127
    Nat.1 < Nat.127
    if Nat.127.is_composite {
        Nat.127.is_composite = exists(b: Nat, c: Nat) {
            Nat.1 < b and Nat.1 < c and Nat.127 = b * c
        }
        let (b: Nat, c: Nat) satisfy { Nat.1 < b and Nat.1 < c and Nat.127 = b * c }
        one_hundred_twenty_seven_composite_contradiction(b, c)
        Nat.1 < b and Nat.1 < c implies not (Nat.127 = b * c)
        not (Nat.127 = b * c)
        Nat.127 = b * c
        false
    }
    not Nat.127.is_composite
    Nat.127.is_prime = Nat.1 < Nat.127 and not Nat.127.is_composite
    Nat.127.is_prime
}

// ---------------------------------------------------------------------------
// (a) The third perfect number: 496 = 2^4 (2^5 - 1).
// ---------------------------------------------------------------------------

/// `2^4 (2^5 - 1) = 496`, since `2^4 = 16` and `2^5 - 1 = 31`.
theorem euclid_product_eq_four_hundred_ninety_six {
    Nat.2.pow(Nat.5 - Nat.1) * (Nat.2.pow(Nat.5) - Nat.1) = Nat.496
} by {
    Nat.5 - Nat.1 = Nat.4
    Nat.2.pow(Nat.5 - Nat.1) = Nat.2.pow(Nat.4)
    two_pow_four
    Nat.2.pow(Nat.4) = Nat.16
    Nat.2.pow(Nat.5 - Nat.1) = Nat.16
    two_pow_5_sub_one
    Nat.2.pow(Nat.5) - Nat.1 = Nat.31
    Nat.2.pow(Nat.5 - Nat.1) * (Nat.2.pow(Nat.5) - Nat.1) = Nat.16 * Nat.31
    nat_16_mul_31
    Nat.16 * Nat.31 = Nat.496
    Nat.2.pow(Nat.5 - Nat.1) * (Nat.2.pow(Nat.5) - Nat.1) = Nat.496
}

/// Four hundred ninety-six is perfect: it is `2^4 (2^5 - 1)` with `2^5 - 1 = 31`
/// prime, so Euclid's construction applies.
theorem four_hundred_ninety_six_is_perfect {
    is_perfect(Nat.496)
} by {
    euclid_construction(Nat.5)
    (Nat.2.pow(Nat.5) - Nat.1).is_prime implies is_perfect(Nat.2.pow(Nat.5 - Nat.1) * (Nat.2.pow(Nat.5) - Nat.1))
    two_pow_five_sub_one_is_prime
    (Nat.2.pow(Nat.5) - Nat.1).is_prime
    is_perfect(Nat.2.pow(Nat.5 - Nat.1) * (Nat.2.pow(Nat.5) - Nat.1))
    euclid_product_eq_four_hundred_ninety_six
    Nat.2.pow(Nat.5 - Nat.1) * (Nat.2.pow(Nat.5) - Nat.1) = Nat.496
    is_perfect(Nat.496)
}

// ---------------------------------------------------------------------------
// (b) The fourth perfect number: 8128 = 2^6 (2^7 - 1).
// ---------------------------------------------------------------------------

/// `2^6 (2^7 - 1) = 8128`, since `2^6 = 64` and `2^7 - 1 = 127`.
theorem euclid_product_eq_eight_thousand_one_hundred_twenty_eight {
    Nat.2.pow(Nat.7 - Nat.1) * (Nat.2.pow(Nat.7) - Nat.1) = Nat.8128
} by {
    Nat.7 - Nat.1 = Nat.6
    Nat.2.pow(Nat.7 - Nat.1) = Nat.2.pow(Nat.6)
    nat_2_pow_6
    Nat.2.pow(Nat.6) = Nat.64
    Nat.2.pow(Nat.7 - Nat.1) = Nat.64
    nat_2_pow_7_sub_one
    Nat.2.pow(Nat.7) - Nat.1 = Nat.127
    Nat.2.pow(Nat.7 - Nat.1) * (Nat.2.pow(Nat.7) - Nat.1) = Nat.64 * Nat.127
    nat_64_mul_127
    Nat.64 * Nat.127 = Nat.8128
    Nat.2.pow(Nat.7 - Nat.1) * (Nat.2.pow(Nat.7) - Nat.1) = Nat.8128
}

/// Eight thousand one hundred twenty-eight is perfect: it is
/// `2^6 (2^7 - 1)` with `2^7 - 1 = 127` prime, so Euclid's construction
/// applies.
theorem eight_thousand_one_hundred_twenty_eight_is_perfect {
    is_perfect(Nat.8128)
} by {
    nat_2_pow_7_sub_one
    Nat.2.pow(Nat.7) - Nat.1 = Nat.127
    one_hundred_twenty_seven_is_prime
    Nat.127.is_prime
    (Nat.2.pow(Nat.7) - Nat.1).is_prime
    euclid_construction(Nat.7)
    (Nat.2.pow(Nat.7) - Nat.1).is_prime implies is_perfect(Nat.2.pow(Nat.7 - Nat.1) * (Nat.2.pow(Nat.7) - Nat.1))
    is_perfect(Nat.2.pow(Nat.7 - Nat.1) * (Nat.2.pow(Nat.7) - Nat.1))
    euclid_product_eq_eight_thousand_one_hundred_twenty_eight
    Nat.2.pow(Nat.7 - Nat.1) * (Nat.2.pow(Nat.7) - Nat.1) = Nat.8128
    is_perfect(Nat.8128)
}

// ---------------------------------------------------------------------------
// (c) Abundant and deficient numbers: twelve is abundant, eight is deficient.
// ---------------------------------------------------------------------------

/// `sigma(3) = 4`, since three is prime.
theorem nat_sigma_three {
    nat_sigma(Nat.3) = Nat.4
} by {
    three_is_prime
    Nat.3.is_prime
    nat_sigma_prime(Nat.3)
    nat_sigma(Nat.3) = Nat.3 + Nat.1
    Nat.3 + Nat.1 = Nat.4
    nat_sigma(Nat.3) = Nat.4
}

/// Four is coprime with three: three is prime and does not divide four.
theorem four_coprime_three {
    Nat.4.coprime(Nat.3)
} by {
    three_is_prime
    Nat.3.is_prime
    gcd_of_prime(Nat.3, Nat.4)
    if Nat.3.gcd(Nat.4) = Nat.1 {
        Nat.3.coprime(Nat.4)
        coprime_comm(Nat.3, Nat.4)
        Nat.4.coprime(Nat.3)
    } else {
        Nat.3.divides(Nat.4)
        nat_three_not_divides_four
        false
    }
}

/// `sigma(12) = sigma(4) sigma(3) = 7 * 4 = 28`, by multiplicativity on the
/// coprime factors four and three.
theorem nat_sigma_twelve {
    nat_sigma(Nat.12) = Nat.28
} by {
    Nat.4 * Nat.3 = Nat.12
    four_coprime_three
    Nat.4.coprime(Nat.3)
    nat_sigma_multiplicative
    is_multiplicative_nat_fn(nat_sigma)
    multiplicative_nat_fn_apply(nat_sigma, Nat.4, Nat.3)
    nat_sigma(Nat.4 * Nat.3) = nat_sigma(Nat.4) * nat_sigma(Nat.3)
    nat_sigma_four
    nat_sigma(Nat.4) = Nat.7
    nat_sigma_three
    nat_sigma(Nat.3) = Nat.4
    nat_sigma(Nat.4 * Nat.3) = Nat.7 * Nat.4
    Nat.7 * Nat.4 = Nat.28
    nat_sigma(Nat.4 * Nat.3) = Nat.28
    Nat.4 * Nat.3 = Nat.12
    nat_sigma(Nat.12) = Nat.28
}

/// `24 + 4 = 28`.
theorem nat_24_add_4 {
    Nat.24 + Nat.4 = Nat.28
} by {
    Nat.24 = Nat.2.read(Nat.4)
    read_add_single(Nat.2, Nat.4, Nat.4)
    Nat.2.read(Nat.4) + Nat.4 = Nat.2.read(Nat.4 + Nat.4)
    Nat.4 + Nat.4 = Nat.8
    Nat.2.read(Nat.4 + Nat.4) = Nat.2.read(Nat.8)
    Nat.2.read(Nat.8) = Nat.28
    Nat.24 + Nat.4 = Nat.28
}

/// `24 < 28`.
theorem nat_24_lt_28 {
    Nat.24 < Nat.28
} by {
    nat_24_add_4
    Nat.24 + Nat.4 = Nat.28
    exists(c: Nat) { Nat.24 + c = Nat.28 }
    Nat.24 <= Nat.28
    lt_ne(Nat.24, Nat.28, Nat.4)
    Nat.24 + Nat.4 = Nat.28
    Nat.4 != Nat.0
    Nat.24 != Nat.28
    Nat.24 < Nat.28
}

/// `2 * 12 = 24`.
theorem two_mul_twelve {
    Nat.2 * Nat.12 = Nat.24
} by {
    mul_two_left(Nat.12)
    Nat.2 * Nat.12 = Nat.12 + Nat.12
    Nat.12 = Nat.1.read(Nat.2)
    read_add_read(Nat.1, Nat.2, Nat.1, Nat.2)
    Nat.1.read(Nat.2) + Nat.1.read(Nat.2) = (Nat.1 + Nat.1).read(Nat.2 + Nat.2)
    Nat.1 + Nat.1 = Nat.2
    Nat.2 + Nat.2 = Nat.4
    (Nat.1 + Nat.1).read(Nat.2 + Nat.2) = Nat.2.read(Nat.4)
    Nat.2.read(Nat.4) = Nat.24
    Nat.12 + Nat.12 = Nat.24
    Nat.2 * Nat.12 = Nat.24
}

/// Twelve is abundant: `sigma(12) = 28` exceeds `2 * 12 = 24`.
theorem twelve_is_abundant {
    is_abundant(Nat.12)
} by {
    is_abundant(Nat.12) = (Nat.2 * Nat.12 < nat_sigma(Nat.12))
    two_mul_twelve
    Nat.2 * Nat.12 = Nat.24
    nat_sigma_twelve
    nat_sigma(Nat.12) = Nat.28
    nat_24_lt_28
    Nat.24 < Nat.28
    Nat.2 * Nat.12 < nat_sigma(Nat.12)
    is_abundant(Nat.12)
}

/// `sigma(8) = 15`: as `8 = 2^3`, the geometric-series relation
/// `(2 - 1) sigma(2^3) + 1 = 2^4` gives `sigma(8) = 15`.
theorem nat_sigma_eight {
    nat_sigma(Nat.8) = Nat.15
} by {
    two_is_prime
    Nat.2.is_prime
    nat_sigma_prime_pow_mult(Nat.2, Nat.3)
    (Nat.2 - Nat.1) * nat_sigma(Nat.2.pow(Nat.3)) + Nat.1 = Nat.2.pow(Nat.3 + Nat.1)
    suc_sub_one(Nat.1)
    Nat.2 - Nat.1 = Nat.1
    Nat.1 * nat_sigma(Nat.2.pow(Nat.3)) + Nat.1 = Nat.2.pow(Nat.3 + Nat.1)
    mul_one_left(nat_sigma(Nat.2.pow(Nat.3)))
    nat_sigma(Nat.2.pow(Nat.3)) + Nat.1 = Nat.2.pow(Nat.3 + Nat.1)
    Nat.3 + Nat.1 = Nat.4
    Nat.2.pow(Nat.3 + Nat.1) = Nat.2.pow(Nat.4)
    two_pow_four
    Nat.2.pow(Nat.4) = Nat.16
    nat_sigma(Nat.2.pow(Nat.3)) + Nat.1 = Nat.16
    two_pow_three
    Nat.2.pow(Nat.3) = Nat.8
    nat_sigma(Nat.8) + Nat.1 = Nat.16
    add_imp_sub(nat_sigma(Nat.8), Nat.1, Nat.16)
    Nat.16 - Nat.1 = nat_sigma(Nat.8)
    Nat.16 - Nat.1 = Nat.15
    nat_sigma(Nat.8) = Nat.15
}

/// `15 < 16`.
theorem nat_15_lt_16 {
    Nat.15 < Nat.16
} by {
    Nat.15 + Nat.1 = Nat.16
    exists(c: Nat) { Nat.15 + c = Nat.16 }
    Nat.15 <= Nat.16
    lt_ne(Nat.15, Nat.16, Nat.1)
    Nat.15 + Nat.1 = Nat.16
    Nat.1 != Nat.0
    Nat.15 != Nat.16
    Nat.15 < Nat.16
}

/// Eight is deficient: `sigma(8) = 15` falls short of `2 * 8 = 16`.
theorem eight_is_deficient {
    is_deficient(Nat.8)
} by {
    is_deficient(Nat.8) = (nat_sigma(Nat.8) < Nat.2 * Nat.8)
    nat_sigma_eight
    nat_sigma(Nat.8) = Nat.15
    nat_mul_2_8
    Nat.2 * Nat.8 = Nat.16
    nat_15_lt_16
    Nat.15 < Nat.16
    nat_sigma(Nat.8) < Nat.2 * Nat.8
    is_deficient(Nat.8)
}

// ---------------------------------------------------------------------------
// (d) Multiples of perfect numbers are abundant.
// ---------------------------------------------------------------------------
//
// Every multiple of a perfect number is abundant: if `sigma(n) = 2 n` and
// `1 < m`, then the divisors `m * d` of `n * m` (for `d | n`), together with
// the divisor `1`, give
//
//     sigma(n * m) >= m * sigma(n) + 1 = 2 n m + 1 > 2 n m,
//
// so `n * m` is abundant.  The general proof is left for future work (it
// needs a sum-over-sublist bound for divisor lists); the instance
// `2 * 6 = 12` is proved here — twelve is abundant by section (c).
//
// theorem multiple_of_perfect_is_abundant(n: Nat, m: Nat) {
//     is_perfect(n) and Nat.1 < m implies is_abundant(n * m)
// }

/// Twice six is twelve.
theorem two_mul_six_eq_twelve {
    Nat.2 * Nat.6 = Nat.12
} by {
    nat_mul_2_6
    Nat.2 * Nat.6 = Nat.12
}

// ---------------------------------------------------------------------------
// (e) The Euclid-Euler theorem.
// ---------------------------------------------------------------------------
//
// A natural number `n` is perfect exactly when it has the form
// `2^(p-1) (2^p - 1)` with `2^p - 1` prime (and then `p` is prime by
// mersenne_prime_imp_prime in mersenne_perfect.ac).  The "if" direction is
// Euclid's construction, proved in mersenne_perfect.ac as
// `euclid_construction` and instantiated here at `p = 5` and `p = 7`.
// The "only if" direction is Euler's theorem: if `n` is even and perfect,
// write `n = 2^(k-1) m` with `m` odd (`k >= 2`); multiplicativity of `sigma`
// on the coprime factors `2^(k-1)` and `m` gives
// `(2^k - 1) sigma(m) = 2^k m`.  Since `2^k - 1` is coprime to `2^k`, it
// divides `m`, say `m = (2^k - 1) r`; substituting and cancelling shows
// `sigma(m) = 2^k r`.  As `r` is a proper divisor of `m` with
// `sigma(m) >= m + r`, equality forces `r = 1`, so `m = 2^k - 1` and
// `sigma(m) = m + 1`, which means `m` is prime.  This direction is left as
// future work (see the sketch at the end of mersenne_perfect.ac).
//
// theorem euclid_euler(n: Nat) {
//     is_perfect(n) and Nat.2.divides(n) implies
//         exists(p: Nat) {
//             (Nat.2.pow(p) - Nat.1).is_prime and
//             n = Nat.2.pow(p - Nat.1) * (Nat.2.pow(p) - Nat.1)
//         }
// }
