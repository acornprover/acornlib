from int import Int, add_neg, add_from_nat
from number_theory.quadratic_residue import Nat, is_quadratic_residue_mod,
    is_unit_quadratic_residue_mod,
    quadratic_residue_coprime_iff_unit,
    quadratic_residue_mul, unit_quadratic_residue_mul,
    prime_coprime_imp_nonzero_congr_mod,
    congr_mod_preserves_coprime,
    unit_quadratic_residue_of_unit_square_congr, square_coprime_imp_base
from number_theory.congruence import congr_mod_refl, congr_mod_symm, congr_mod_trans,
    congr_mod_pow, mod_congr_mod_self, mod_add_mul,
    divides_of_congr_mod_zero, congr_mod_zero_of_divides
from number_theory.totient import congr_mod_add_cancel_right_pos, congr_mod_below_eq,
    coprime_below_prime
from number_theory.fermat import prime_divides_mul
from number_theory.legendre_symbol import legendre_symbol, is_quadratic_nonresidue_mod,
    is_unit_quadratic_nonresidue_mod, unit_quadratic_nonresidue_is_nonresidue,
    legendre_symbol_one_of_prime_unit_quadratic_residue,
    legendre_symbol_neg_one_of_prime_unit_nonresidue
from number_theory.quadratic_residue_supplements import prime_pred_legendre_symbol_by_half_parity,
    prime_two_legendre_symbol_mod_eight
from number_theory.primitive_root import is_order_double_unit_generator_mod
from number_theory.mobius_sums import int_sum_map_neg
from nat import add_sub, add_imp_sub, sq_eq_mul, mul_two_left,
    add_cancels_left, add_one_right, lte_trans, lte_mul_both, lte_mul, sum_lte, lte_and_lt,
    lt_suc, lt_add_left, sub_pos, lt_imp_lte_suc, trichotomy,
    lt_and_lte, small_mod, mod_lt, divides_self, divides_mul,
    alt_suc_ne_zero, sub_lt, pos_of_ne_zero, sub_one_lt, mod_of_zero, lt_trans,
    cross_sum_lte, lte_cancel_suc, alt_induction, div_mul
from list import List, map, sum, length_range, map_length, map_contains, map_contains_of_contains,
    range_contains_iff_lt, range_is_unique, filter_preserves_unique, filter_equivalent_to_and,
    sum_map_of_pointwise, sum_add, map_add, unique_list_sum,
    singleton_unique, is_permutation,
    unique_same_contains_imp_permutation,
    permutation_preserves_length, injective_map_is_unique,
    add_contains_or, unique_same_contains_map_sum_eq
from data.basic.functions import is_injective_fn
numerals Nat
numerals Int

// ---------------------------------------------------------------------------
// Applications of the Legendre symbol.
//
// The classical consequences of the Legendre-symbol API: the product formula
// `sum_{a=1}^{p-1} (a/p) = 0`, the count of the unit quadratic residues modulo
// an odd prime, and the two supplements restated.
// ---------------------------------------------------------------------------

/// The square of the successor, reduced modulo `p`.
define square_suc_mod(p: Nat, x: Nat) -> Nat {
    (x.suc * x.suc).mod(p)
}

/// Unit quadratic-residue status at a fixed modulus, as a function of the
/// target.
define is_unit_qr_at(p: Nat, a: Nat) -> Bool {
    is_unit_quadratic_residue_mod(a, p)
}

/// Unit quadratic-nonresidue status at a fixed modulus, as a function of the
/// target.
define is_unit_nonqr_at(p: Nat, a: Nat) -> Bool {
    is_unit_quadratic_nonresidue_mod(a, p)
}

/// The Legendre symbol at a fixed modulus, as a function of the target.
define legendre_at(p: Nat, a: Nat) -> Int {
    legendre_symbol(a, p)
}

/// The positive representatives `1, ..., p-1` of the nonzero residues modulo `p`.
define positive_residues(p: Nat) -> List[Nat] {
    map((p - Nat.1).range, Nat.suc)
}

/// The squares of the positive half `1, ..., h` reduced modulo `p`, where the
/// odd prime `p` is `2 * h + 1`.
define half_square_residues(p: Nat, h: Nat) -> List[Nat] {
    map(h.range, square_suc_mod(p))
}

/// The unit quadratic residues in `1, ..., p-1`.
define unit_qr_positive_list(p: Nat) -> List[Nat] {
    positive_residues(p).filter(is_unit_qr_at(p))
}

/// The unit quadratic nonresidues in `1, ..., p-1`.
define unit_nonqr_positive_list(p: Nat) -> List[Nat] {
    positive_residues(p).filter(is_unit_nonqr_at(p))
}

/// The sum of the Legendre symbols over the positive residues `1, ..., p-1`.
define legendre_symbol_sum_positive(p: Nat) -> Int {
    sum(map(positive_residues(p), legendre_at(p)))
}

/// The constant-one summand.
define int_one_summand(a: Nat) -> Int {
    Int.1
}






/// The constant-minus-one summand.
define int_neg_one_summand(a: Nat) -> Int {
    -Int.1
}

// ---------------------------------------------------------------------------
// Sums of constant functions.
// ---------------------------------------------------------------------------

/// The sum of the constant one over a list is its length, as an integer.
theorem int_sum_const_one(l: List[Nat]) {
    sum(map(l, int_one_summand)) = Int.from_nat(l.length)
} by {
    define p(xs: List[Nat]) -> Bool {
        sum(map(xs, int_one_summand)) = Int.from_nat(xs.length)
    }
    map(List.nil[Nat], int_one_summand) = List.nil[Int]
    sum(List.nil[Int]) = Int.0
    Int.from_nat(Nat.0) = Int.0
    List.nil[Nat].length = Nat.0
    sum(map(List.nil[Nat], int_one_summand)) = Int.from_nat(List.nil[Nat].length)
    p(List.nil[Nat])
    forall(head: Nat, tail: List[Nat]) {
        if p(tail) {
            sum(map(tail, int_one_summand)) = Int.from_nat(tail.length)
            map(List.cons(head, tail), int_one_summand) =
                List.cons(int_one_summand(head), map(tail, int_one_summand))
            int_one_summand(head) = Int.1
            sum(List.cons(Int.1, map(tail, int_one_summand))) =
                Int.1 + sum(map(tail, int_one_summand))
            sum(map(List.cons(head, tail), int_one_summand)) =
                Int.1 + sum(map(tail, int_one_summand))
            Int.1 + sum(map(tail, int_one_summand)) =
                Int.1 + Int.from_nat(tail.length)
            Int.1 = Int.from_nat(Nat.1)
            Int.from_nat(Nat.1) + Int.from_nat(tail.length) =
                Int.from_nat(Nat.1 + tail.length)
            Int.1 + Int.from_nat(tail.length) =
                Int.from_nat(Nat.1 + tail.length)
            Nat.1 + tail.length = tail.length.suc
            Int.from_nat(Nat.1 + tail.length) = Int.from_nat(tail.length.suc)
            List.cons(head, tail).length = tail.length.suc
            Int.from_nat(tail.length.suc) = Int.from_nat(List.cons(head, tail).length)
            sum(map(List.cons(head, tail), int_one_summand)) =
                Int.from_nat(List.cons(head, tail).length)
            p(List.cons(head, tail))
        }
    }
    forall(head: Nat, tail: List[Nat]) {
        p(tail) implies p(List.cons(head, tail))
    }
    p(List.nil[Nat]) and forall(head: Nat, tail: List[Nat]) {
        p(tail) implies p(List.cons(head, tail))
    }
    List.induction(p)
    forall(xs: List[Nat]) { p(xs) }
    p(l)
}

/// The sum of the constant minus one over a list is the negation of its
/// length, as an integer.
theorem int_sum_const_neg_one(l: List[Nat]) {
    sum(map(l, int_neg_one_summand)) = -(Int.from_nat(l.length))
} by {
    forall(x: Nat) {
        int_neg_one_summand(x) = -(int_one_summand(x))
    }
    sum_map_of_pointwise(l, int_neg_one_summand,
        function(x: Nat) { -(int_one_summand(x)) })
    sum(map(l, int_neg_one_summand)) = sum(map(l, function(x: Nat) { -(int_one_summand(x)) }))
    int_sum_map_neg(l, int_one_summand)
    sum(map(l, function(x: Nat) { -(int_one_summand(x)) })) = -sum(map(l, int_one_summand))
    int_sum_const_one(l)
    sum(map(l, int_one_summand)) = Int.from_nat(l.length)
    sum(map(l, int_neg_one_summand)) = -(Int.from_nat(l.length))
}


// ---------------------------------------------------------------------------
// Difference of squares and congruence-to-divisibility helpers.
// ---------------------------------------------------------------------------

/// The difference of squares factors: `(a - b)(a + b) = a^2 - b^2` when
/// `b <= a`.
theorem nat_sq_diff(a: Nat, b: Nat) {
    b <= a implies (a - b) * (a + b) = a * a - b * b
} by {
    if b <= a {
        add_sub(a, b)
        a - b + b = a
        (a - b + b) * (a - b + b) = a * a
        (a - b + b) * (a - b + b) = (a - b) * (a - b + b) + b * (a - b + b)
        (a - b) * (a - b + b) + b * (a - b + b) = (a - b) * (a - b) + (a - b) * b + b * (a - b) + b * b
        (a - b) * (a - b + b) = (a - b) * (a - b) + (a - b) * b
        b * (a - b + b) = b * (a - b) + b * b
        (a - b) * (a - b) + (a - b) * b + b * (a - b) + b * b = a * a
        (a - b) * (a + b) + b * b = (a - b) * (a - b) + (a - b) * b + b * (a - b) + b * b
        add_imp_sub((a - b) * (a + b), b * b, a * a)
        a * a - b * b = (a - b) * (a + b)
        (a - b) * (a + b) = a * a - b * b
    }
}

/// Subtraction never exceeds its minuend.
theorem sub_le_self(a: Nat, b: Nat) {
    a - b <= a
} by {
    if a < b {
        sub_lt(a, b)
        a - b = Nat.0
        Nat.0 <= a
        a - b <= a
    } else {
        add_sub(a, b)
        a - b + b = a
        a - b <= a - b + b
        a - b <= a
    }
}

/// Congruence with the smaller number on the right gives divisibility of the
/// difference.
theorem congr_le_imp_divides_diff(a: Nat, b: Nat, n: Nat) {
    n != Nat.0 and b <= a and a.congr_mod(b, n) implies n.divides(a - b)
} by {
    if n != Nat.0 and b <= a and a.congr_mod(b, n) {
        add_sub(a, b)
        a - b + b = a
        congr_mod_refl(b, n)
        b.congr_mod(b, n)
        congr_mod_trans(a - b + b, a, b, n)
        (a - b + b).congr_mod(b, n)
        Nat.0 + b = b
        congr_mod_trans((a - b + b), b, Nat.0 + b, n)
        (a - b + b).congr_mod(Nat.0 + b, n)
        congr_mod_add_cancel_right_pos(a - b, Nat.0, b, n)
        (a - b).congr_mod(Nat.0, n)
        divides_of_congr_mod_zero(n, a - b)
        n.divides(a - b)
    }
}

/// Divisibility of a difference with the smaller number on the right gives
/// congruence.
theorem divides_diff_imp_congr(a: Nat, b: Nat, n: Nat) {
    n != Nat.0 and b <= a and n.divides(a - b) implies a.congr_mod(b, n)
} by {
    if n != Nat.0 and b <= a and n.divides(a - b) {
        n.divides(a - b) = exists(q: Nat) { n * q = a - b }
        let k: Nat satisfy { n * k = a - b }
        add_sub(a, b)
        a - b + b = a
        n * k + b = a
        mod_add_mul(k, n, b)
        (k * n + b).mod(n) = b.mod(n)
        k * n = n * k
        (n * k + b).mod(n) = b.mod(n)
        a.mod(n) = b.mod(n)
        a.congr_mod(b, n)
    }
}

/// A number below a positive modulus that it divides must be zero.
theorem divides_below_imp_zero(d: Nat, p: Nat) {
    p != Nat.0 and d < p and p.divides(d) implies d = Nat.0
} by {
    if p != Nat.0 and d < p and p.divides(d) {
        congr_mod_zero_of_divides(p, d)
        d.congr_mod(Nat.0, p)
        congr_mod_below_eq(p, d, Nat.0)
        d = Nat.0
    }
}

// ---------------------------------------------------------------------------
// The square map on the positive half of an odd prime.
// ---------------------------------------------------------------------------

/// Squaring is injective on the positive half `1, ..., h` of an odd prime
/// `p = 2 * h + 1`.
theorem square_half_injective(p: Nat, h: Nat, x: Nat, y: Nat) {
    p.is_prime and p = Nat.2 * h + Nat.1 and Nat.0 < x and x <= h and Nat.0 < y and y <= h
        and (x * x).mod(p) = (y * y).mod(p)
        implies x = y
} by {
    if p.is_prime and p = Nat.2 * h + Nat.1 and Nat.0 < x and x <= h and Nat.0 < y and y <= h
            and (x * x).mod(p) = (y * y).mod(p) {
        (x * x).congr_mod(y * y, p)
        add_one_right(Nat.2 * h)
        p = (Nat.2 * h).suc
        alt_suc_ne_zero(Nat.2 * h)
        (Nat.2 * h).suc != Nat.0
        p != Nat.0
        pos_of_ne_zero(p)
        Nat.0 < p
        lt_suc(Nat.2 * h)
        Nat.2 * h < p
        mul_two_left(h)
        Nat.2 * h = h + h
        sum_lte(x, y, h, h)
        x + y <= h + h
        x + y <= Nat.2 * h
        lte_and_lt(x + y, Nat.2 * h, p)
        x + y < p
        lte_mul(h, Nat.2)
        h <= h * Nat.2
        h * Nat.2 = Nat.2 * h
        h <= Nat.2 * h
        lte_and_lt(h, Nat.2 * h, p)
        h < p
        lte_and_lt(y, h, p)
        y < p
        lte_and_lt(x, h, p)
        x < p
        if y < x {
            sub_pos(x, y)
            x - y > Nat.0
            sub_le_self(x, y)
            x - y <= x
            lte_and_lt(x - y, x, p)
            x - y < p
            lt_imp_lte_suc(y, x)
            y.suc <= x
            lte_trans(y, y.suc, x)
            y <= x
            lte_mul_both(y, y, x)
            y * y <= y * x
            lte_mul_both(x, y, x)
            x * y <= x * x
            y * x = x * y
            y * y <= y * x
            lte_trans(y * y, y * x, x * x)
            y * y <= x * x
            nat_sq_diff(x, y)
            (x - y) * (x + y) = x * x - y * y
            congr_le_imp_divides_diff(x * x, y * y, p)
            p.divides(x * x - y * y)
            p.divides((x - y) * (x + y))
            prime_divides_mul(p, x - y, x + y)
            p.divides(x - y) or p.divides(x + y)
            if p.divides(x + y) {
                divides_below_imp_zero(x + y, p)
                x + y = Nat.0
                lt_add_left(x, Nat.0, y)
                Nat.0 < y
                x < x + y
                lte_and_lt(Nat.0, x, x + y)
                Nat.0 < x + y
                false
            }
            divides_below_imp_zero(x - y, p)
            x - y = Nat.0
            false
        }
        if x < y {
            sub_pos(y, x)
            y - x > Nat.0
            sub_le_self(y, x)
            y - x <= y
            lte_and_lt(y - x, y, p)
            y - x < p
            lt_imp_lte_suc(x, y)
            x.suc <= y
            lte_trans(x, x.suc, y)
            x <= y
            lte_mul_both(x, x, y)
            x * x <= x * y
            lte_mul_both(y, x, y)
            y * x <= y * y
            x * y = y * x
            x * x <= x * y
            lte_trans(x * x, x * y, y * y)
            x * x <= y * y
            nat_sq_diff(y, x)
            (y - x) * (y + x) = y * y - x * x
            congr_mod_symm(x * x, y * y, p)
            (y * y).congr_mod(x * x, p)
            congr_le_imp_divides_diff(y * y, x * x, p)
            p.divides(y * y - x * x)
            p.divides((y - x) * (y + x))
            prime_divides_mul(p, y - x, y + x)
            p.divides(y - x) or p.divides(y + x)
            if p.divides(y + x) {
                divides_below_imp_zero(y + x, p)
                y + x = Nat.0
                false
            }
            divides_below_imp_zero(y - x, p)
            y - x = Nat.0
            false
        }
        trichotomy(x, y)
        x = y
    }
}

/// A locally injective map on a range gives a unique list.
theorem map_range_injective_is_unique(n: Nat, f: Nat -> Nat) {
    (forall(x: Nat, y: Nat) { x < n and y < n and f(x) = f(y) implies x = y })
        implies map(n.range, f).is_unique
} by {
    if forall(x: Nat, y: Nat) { x < n and y < n and f(x) = f(y) implies x = y } {
        define pred(m: Nat) -> Bool {
            (forall(x: Nat, y: Nat) { x < m and y < m and f(x) = f(y) implies x = y })
                implies map(m.range, f).is_unique
        }
        map(Nat.0.range, f) = map(List.nil[Nat], f)
        map(List.nil[Nat], f) = List.nil[Nat]
        List.nil[Nat].is_unique
        pred(Nat.0)
        forall(m: Nat) {
            if pred(m) {
                if forall(x: Nat, y: Nat) { x < m.suc and y < m.suc and f(x) = f(y) implies x = y } {
                    forall(x: Nat, y: Nat) {
                        if x < m and y < m and f(x) = f(y) {
                            lt_trans(x, m, m.suc)
                            x < m.suc
                            lt_trans(y, m, m.suc)
                            y < m.suc
                            x < m.suc and y < m.suc and f(x) = f(y)
                            x = y
                        }
                    }
                    pred(m) =
                        ((forall(x: Nat, y: Nat) {
                            x < m and y < m and f(x) = f(y) implies x = y
                        }) implies map(m.range, f).is_unique)
                    forall(x: Nat, y: Nat) {
                        x < m and y < m and f(x) = f(y) implies x = y
                    }
                    map(m.range, f).is_unique
                    m.suc.range = m.range.append(m)
                    m.range.append(m) = m.range + List.singleton(m)
                    map(m.range + List.singleton(m), f) = map(m.range, f) + map(List.singleton(m), f)
                    map(List.singleton(m), f) = List.singleton(f(m))
                    map(m.suc.range, f) = map(m.range, f) + List.singleton(f(m))
                    singleton_unique(f(m))
                    forall(y: Nat) {
                        if List.singleton(f(m)).contains(y) {
                            y = f(m)
                            if map(m.range, f).contains(y) {
                                map_contains(m.range, f, y)
                                let x: Nat satisfy { m.range.contains(x) and f(x) = y }
                                m.range.contains(x) = x < m
                                x < m
                                lt_trans(x, m, m.suc)
                                x < m.suc
                                lt_suc(m)
                                m < m.suc
                                x < m.suc and m < m.suc and f(x) = f(m)
                                x = m
                                false
                            }
                            not map(m.range, f).contains(y)
                        }
                        not (map(m.range, f).contains(y) and List.singleton(f(m)).contains(y))
                    }
                    unique_list_sum(map(m.range, f), List.singleton(f(m)))
                    (map(m.range, f) + List.singleton(f(m))).is_unique
                    map(m.suc.range, f).is_unique
                }
                pred(m.suc)
            }
        }
        forall(m: Nat) {
            pred(m) implies pred(m.suc)
        }
        pred(Nat.0) and forall(m: Nat) {
            pred(m) implies pred(m.suc)
        }
        alt_induction(pred)
        forall(m: Nat) { pred(m) }
        pred(n) =
            ((forall(x: Nat, y: Nat) { x < n and y < n and f(x) = f(y) implies x = y })
                implies map(n.range, f).is_unique)
        forall(x: Nat, y: Nat) { x < n and y < n and f(x) = f(y) implies x = y }
        map(n.range, f).is_unique
    }
}

/// The squares of `1, ..., h` are distinct modulo the odd prime `p = 2h+1`.
theorem half_square_residues_unique(p: Nat, h: Nat) {
    p.is_prime and p = Nat.2 * h + Nat.1 implies half_square_residues(p, h).is_unique
} by {
    if p.is_prime and p = Nat.2 * h + Nat.1 {
        forall(x: Nat, y: Nat) {
            if x < h and y < h and square_suc_mod(p, x) = square_suc_mod(p, y) {
                alt_suc_ne_zero(x)
                x.suc != Nat.0
                pos_of_ne_zero(x.suc)
                Nat.0 < x.suc
                lt_imp_lte_suc(x, h)
                x.suc <= h
                alt_suc_ne_zero(y)
                y.suc != Nat.0
                pos_of_ne_zero(y.suc)
                Nat.0 < y.suc
                lt_imp_lte_suc(y, h)
                y.suc <= h
                square_half_injective(p, h, x.suc, y.suc)
                x.suc = y.suc
                x = y
            }
        }
        map_range_injective_is_unique(h, square_suc_mod(p))
        map(h.range, square_suc_mod(p)).is_unique
        half_square_residues(p, h).is_unique
    }
}

/// Every square of an element of the positive half is a unit quadratic residue
/// in `1, ..., p-1`.
theorem half_square_residues_member_unit_qr(p: Nat, h: Nat, v: Nat) {
    p.is_prime and p = Nat.2 * h + Nat.1 and half_square_residues(p, h).contains(v)
        implies is_unit_quadratic_residue_mod(v, p)
} by {
    if p.is_prime and p = Nat.2 * h + Nat.1 and half_square_residues(p, h).contains(v) {
        map_contains(h.range, square_suc_mod(p), v)
        let x: Nat satisfy {
            h.range.contains(x) and square_suc_mod(p, x) = v
        }
        h.range.contains(x) = x < h
        x < h
        alt_suc_ne_zero(x)
        x.suc != Nat.0
        pos_of_ne_zero(x.suc)
        Nat.0 < x.suc
        lt_imp_lte_suc(x, h)
        x.suc <= h
        lt_imp_lte_suc(Nat.0, x.suc)
        Nat.1 <= x.suc
        x.suc <= h
        add_one_right(Nat.2 * h)
        p = (Nat.2 * h).suc
        lt_suc(Nat.2 * h)
        Nat.2 * h < p
        lte_mul(h, Nat.2)
        h <= h * Nat.2
        h * Nat.2 = Nat.2 * h
        h <= Nat.2 * h
        lte_and_lt(h, Nat.2 * h, p)
        h < p
        lte_and_lt(x.suc, h, p)
        x.suc < p
        coprime_below_prime(p, x.suc)
        x.suc.coprime(p)
        congr_mod_refl(x.suc * x.suc, p)
        (x.suc * x.suc).congr_mod(x.suc * x.suc, p)
        mod_congr_mod_self(x.suc * x.suc, p)
        (x.suc * x.suc).mod(p).congr_mod(x.suc * x.suc, p)
        congr_mod_symm((x.suc * x.suc).mod(p), x.suc * x.suc, p)
        (x.suc * x.suc).congr_mod((x.suc * x.suc).mod(p), p)
        (x.suc * x.suc).congr_mod(v, p)
        unit_quadratic_residue_of_unit_square_congr(x.suc, v, p)
        is_unit_quadratic_residue_mod(v, p)
    }
}

/// Every square of an element of the positive half lies in `1, ..., p-1`.
theorem half_square_residues_member_below_prime(p: Nat, h: Nat, v: Nat) {
    p.is_prime and p = Nat.2 * h + Nat.1 and half_square_residues(p, h).contains(v)
        implies Nat.0 < v and v < p
} by {
    if p.is_prime and p = Nat.2 * h + Nat.1 and half_square_residues(p, h).contains(v) {
        map_contains(h.range, square_suc_mod(p), v)
        let x: Nat satisfy {
            h.range.contains(x) and square_suc_mod(p, x) = v
        }
        h.range.contains(x) = x < h
        x < h
        alt_suc_ne_zero(x)
        x.suc != Nat.0
        pos_of_ne_zero(x.suc)
        Nat.0 < x.suc
        lt_imp_lte_suc(x, h)
        x.suc <= h
        lt_imp_lte_suc(Nat.0, x.suc)
        Nat.1 <= x.suc
        x.suc <= h
        add_one_right(Nat.2 * h)
        p = (Nat.2 * h).suc
        lt_suc(Nat.2 * h)
        Nat.2 * h < p
        lte_mul(h, Nat.2)
        h <= h * Nat.2
        h * Nat.2 = Nat.2 * h
        h <= Nat.2 * h
        lte_and_lt(h, Nat.2 * h, p)
        h < p
        lte_and_lt(x.suc, h, p)
        x.suc < p
        coprime_below_prime(p, x.suc)
        x.suc.coprime(p)
        square_coprime_imp_base(x.suc, p)
        (x.suc * x.suc).coprime(p)
        mod_congr_mod_self(x.suc * x.suc, p)
        (x.suc * x.suc).mod(p).congr_mod(x.suc * x.suc, p)
        congr_mod_symm((x.suc * x.suc).mod(p), x.suc * x.suc, p)
        (x.suc * x.suc).congr_mod((x.suc * x.suc).mod(p), p)
        congr_mod_preserves_coprime(x.suc * x.suc, (x.suc * x.suc).mod(p), p)
        (x.suc * x.suc).mod(p).coprime(p)
        square_suc_mod(p, x) = v
        v.coprime(p)
        prime_coprime_imp_nonzero_congr_mod(p, v)
        not v.congr_mod(Nat.0, p)
        add_one_right(Nat.2 * h)
        p = (Nat.2 * h).suc
        alt_suc_ne_zero(Nat.2 * h)
        (Nat.2 * h).suc != Nat.0
        p != Nat.0
        mod_lt(x.suc * x.suc, p)
        (x.suc * x.suc).mod(p) < p
        v < p
        if v = Nat.0 {
            mod_of_zero(p)
            v.mod(p) = Nat.0
            Nat.0.congr_mod(Nat.0, p)
            v.congr_mod(Nat.0, p)
            false
        }
        v != Nat.0
        pos_of_ne_zero(v)
        Nat.0 < v
        Nat.0 < v and v < p
    }
}

/// Every unit quadratic residue in `1, ..., p-1` is the square of some element
/// of the positive half `1, ..., h`.
theorem square_residues_cover_unit_qr(p: Nat, h: Nat, a: Nat) {
    p.is_prime and p = Nat.2 * h + Nat.1 and Nat.0 < a and a < p
        and is_unit_quadratic_residue_mod(a, p)
        implies exists(i: Nat) { i < h and square_suc_mod(p, i) = a }
} by {
    if p.is_prime and p = Nat.2 * h + Nat.1 and Nat.0 < a and a < p
            and is_unit_quadratic_residue_mod(a, p) {
        add_one_right(Nat.2 * h)
        p = (Nat.2 * h).suc
        alt_suc_ne_zero(Nat.2 * h)
        (Nat.2 * h).suc != Nat.0
        p != Nat.0
        is_unit_quadratic_residue_mod(a, p) =
            exists(x: Nat) { x.coprime(p) and x.pow(Nat.2).congr_mod(a, p) }
        let x: Nat satisfy { x.coprime(p) and x.pow(Nat.2).congr_mod(a, p) }
        let r: Nat = x.mod(p)
        mod_lt(x, p)
        r < p
        mod_congr_mod_self(x, p)
        r.congr_mod(x, p)
        congr_mod_pow(r, x, p, Nat.2)
        r.pow(Nat.2).congr_mod(x.pow(Nat.2), p)
        congr_mod_trans(r.pow(Nat.2), x.pow(Nat.2), a, p)
        r.pow(Nat.2).congr_mod(a, p)
        sq_eq_mul(r)
        r.pow(Nat.2) = r * r
        (r * r).congr_mod(a, p)
        congr_mod_symm(r, x, p)
        x.congr_mod(r, p)
        congr_mod_preserves_coprime(x, r, p)
        r.coprime(p)
        prime_coprime_imp_nonzero_congr_mod(p, r)
        not r.congr_mod(Nat.0, p)
        if r = Nat.0 {
            mod_of_zero(p)
            r.mod(p) = Nat.0
            Nat.0.congr_mod(Nat.0, p)
            r.congr_mod(Nat.0, p)
            false
        }
        r != Nat.0
        pos_of_ne_zero(r)
        Nat.0 < r
        if r <= h {
            sub_one_lt(r)
            r - Nat.1 < r
            lt_and_lte(r - Nat.1, r, h)
            r - Nat.1 < h
            add_sub(r, Nat.1)
            r - Nat.1 + Nat.1 = r
            add_one_right(r - Nat.1)
            (r - Nat.1).suc = r - Nat.1 + Nat.1
            (r - Nat.1).suc = r
            (r * r).mod(p) = a.mod(p)
            small_mod(a, p)
            a.mod(p) = a
            (r * r).mod(p) = a
            (r - Nat.1).suc * (r - Nat.1).suc = r * r
            square_suc_mod(p, r - Nat.1) = a
            exists(i: Nat) { i < h and square_suc_mod(p, i) = a }
        } else {
            let s: Nat = p - r
            sub_pos(p, r)
            p - r > Nat.0
            s > Nat.0
            add_sub(p, r)
            p - r + r = p
            s + r = p
            lt_imp_lte_suc(h, r)
            h.suc <= r
            add_one_right(h)
            h.suc = h + Nat.1
            h + Nat.1 <= r
            lte_trans(h, h + Nat.1, r)
            h <= r
            (h + Nat.1) + h = Nat.2 * h + Nat.1
            r + s = s + r
            (h + Nat.1) + h = r + s
            cross_sum_lte(h + Nat.1, h, r, s)
            s <= h
            lt_imp_lte_suc(Nat.0, s)
            Nat.1 <= s
            lte_trans(s, h, r)
            s <= r
            nat_sq_diff(r, s)
            (r - s) * (r + s) = r * r - s * s
            divides_self(p)
            p.divides(p)
            divides_mul(p, r - s, p)
            p.divides((r - s) * p)
            r + s = p
            (r - s) * (r + s) = (r - s) * p
            p.divides(r * r - s * s)
            lte_mul_both(s, s, r)
            s * s <= s * r
            lte_mul_both(r, s, r)
            r * s <= r * r
            s * r = r * s
            s * s <= s * r
            lte_trans(s * s, s * r, r * r)
            s * s <= r * r
            divides_diff_imp_congr(r * r, s * s, p)
            (r * r).congr_mod(s * s, p)
            congr_mod_symm(r * r, s * s, p)
            (s * s).congr_mod(r * r, p)
            congr_mod_trans(s * s, r * r, a, p)
            (s * s).congr_mod(a, p)
            sub_one_lt(s)
            s - Nat.1 < s
            lt_and_lte(s - Nat.1, s, h)
            s - Nat.1 < h
            add_sub(s, Nat.1)
            s - Nat.1 + Nat.1 = s
            add_one_right(s - Nat.1)
            (s - Nat.1).suc = s - Nat.1 + Nat.1
            (s - Nat.1).suc = s
            (s * s).mod(p) = a.mod(p)
            small_mod(a, p)
            a.mod(p) = a
            (s * s).mod(p) = a
            (s - Nat.1).suc * (s - Nat.1).suc = s * s
            square_suc_mod(p, s - Nat.1) = a
            exists(j: Nat) { j < h and square_suc_mod(p, j) = a }
        }
    }
}

// ---------------------------------------------------------------------------
// Membership in the lists of positive residues.
// ---------------------------------------------------------------------------

/// The positive residues are unique.
theorem positive_residues_unique(p: Nat) {
    positive_residues(p).is_unique
} by {
    forall(x: Nat, y: Nat) {
        x.suc = y.suc implies x = y
    }
    is_injective_fn(Nat.suc)
    range_is_unique(p - Nat.1)
    injective_map_is_unique((p - Nat.1).range, Nat.suc)
    map((p - Nat.1).range, Nat.suc).is_unique
    positive_residues(p).is_unique
}

/// Membership in the positive residues means lying in `1, ..., p-1`.
theorem positive_residues_contains_iff(p: Nat, a: Nat) {
    positive_residues(p).contains(a) = (Nat.0 < a and a < p)
} by {
    if positive_residues(p).contains(a) {
        map_contains((p - Nat.1).range, Nat.suc, a)
        let x: Nat satisfy { (p - Nat.1).range.contains(x) and x.suc = a }
        (p - Nat.1).range.contains(x) = x < p - Nat.1
        x < p - Nat.1
        lt_imp_lte_suc(x, p - Nat.1)
        x.suc <= p - Nat.1
        x.suc = a
        a <= p - Nat.1
        if p = Nat.0 {
            sub_lt(p, Nat.1)
            p - Nat.1 = Nat.0
            a <= Nat.0
            a = Nat.0
            Nat.0 < a
            false
        }
        p != Nat.0
        pos_of_ne_zero(p)
        Nat.0 < p
        sub_one_lt(p)
        p - Nat.1 < p
        lte_and_lt(a, p - Nat.1, p)
        a < p
        alt_suc_ne_zero(x)
        x.suc != Nat.0
        a != Nat.0
        pos_of_ne_zero(a)
        Nat.0 < a
        Nat.0 < a and a < p
    }
    if Nat.0 < a and a < p {
        pos_of_ne_zero(a)
        a != Nat.0
        lt_imp_lte_suc(a, p)
        a.suc <= p
        add_one_right(a)
        a + Nat.1 = a.suc
        a + Nat.1 <= p
        lte_and_lt(Nat.1, a, p)
        Nat.1 < p
        lt_imp_lte_suc(Nat.1, p)
        Nat.1.suc <= p
        Nat.1 <= p
        add_sub(p, Nat.1)
        p - Nat.1 + Nat.1 = p
        add_one_right(p - Nat.1)
        (p - Nat.1).suc = p - Nat.1 + Nat.1
        (p - Nat.1).suc = p
        a + Nat.1 <= (p - Nat.1).suc
        a.suc <= (p - Nat.1).suc
        lte_cancel_suc(a, p - Nat.1)
        a <= p - Nat.1
        sub_one_lt(a)
        a - Nat.1 < a
        lt_and_lte(a - Nat.1, a, p - Nat.1)
        a - Nat.1 < p - Nat.1
        range_contains_iff_lt(p - Nat.1, a - Nat.1)
        (p - Nat.1).range.contains(a - Nat.1) = (a - Nat.1 < p - Nat.1)
        (p - Nat.1).range.contains(a - Nat.1)
        add_sub(a, Nat.1)
        a - Nat.1 + Nat.1 = a
        add_one_right(a - Nat.1)
        (a - Nat.1).suc = a - Nat.1 + Nat.1
        (a - Nat.1).suc = a
        map_contains_of_contains((p - Nat.1).range, Nat.suc, a - Nat.1)
        map((p - Nat.1).range, Nat.suc).contains(a)
        positive_residues(p).contains(a)
    }
    (positive_residues(p).contains(a) = (Nat.0 < a and a < p)) = true
}

/// The unit quadratic residues in `1, ..., p-1` are exactly the squares of the
/// positive half.
theorem unit_qr_positive_membership_iff_half_square(p: Nat, h: Nat, v: Nat) {
    p.is_prime and p = Nat.2 * h + Nat.1 implies (unit_qr_positive_list(p).contains(v) = half_square_residues(p, h).contains(v))
} by {
    if p.is_prime and p = Nat.2 * h + Nat.1 {
        if unit_qr_positive_list(p).contains(v) {
            filter_equivalent_to_and(positive_residues(p), is_unit_qr_at(p), v)
            unit_qr_positive_list(p).contains(v) =
                (positive_residues(p).contains(v) and is_unit_qr_at(p, v))
            positive_residues(p).contains(v)
            is_unit_qr_at(p, v)
            is_unit_quadratic_residue_mod(v, p)
            positive_residues_contains_iff(p, v)
            positive_residues(p).contains(v) = (Nat.0 < v and v < p)
            Nat.0 < v
            v < p
            square_residues_cover_unit_qr(p, h, v)
            exists(i: Nat) { i < h and square_suc_mod(p, i) = v }
            let i: Nat satisfy { i < h and square_suc_mod(p, i) = v }
            i < h
            square_suc_mod(p, i) = v
            range_contains_iff_lt(h, i)
            h.range.contains(i) = (i < h)
            h.range.contains(i)
            map_contains_of_contains(h.range, square_suc_mod(p), i)
            map(h.range, square_suc_mod(p)).contains(square_suc_mod(p, i))
            square_suc_mod(p, i) = v
            map(h.range, square_suc_mod(p)).contains(v)
            half_square_residues(p, h).contains(v)
        }
        if half_square_residues(p, h).contains(v) {
            half_square_residues_member_unit_qr(p, h, v)
            is_unit_quadratic_residue_mod(v, p)
            half_square_residues_member_below_prime(p, h, v)
            Nat.0 < v
            v < p
            positive_residues_contains_iff(p, v)
            positive_residues(p).contains(v) = (Nat.0 < v and v < p)
            positive_residues(p).contains(v)
            is_unit_qr_at(p, v)
            filter_equivalent_to_and(positive_residues(p), is_unit_qr_at(p), v)
            unit_qr_positive_list(p).contains(v) =
                (positive_residues(p).contains(v) and is_unit_qr_at(p, v))
            unit_qr_positive_list(p).contains(v)
        }
        (unit_qr_positive_list(p).contains(v) = half_square_residues(p, h).contains(v)) = true
    }
}

/// The unit quadratic residues in `1, ..., p-1` form a permutation of the
/// squares of the positive half.
theorem unit_qr_positive_permutation_half_squares(p: Nat, h: Nat) {
    p.is_prime and p = Nat.2 * h + Nat.1
        implies is_permutation(unit_qr_positive_list(p), half_square_residues(p, h))
} by {
    if p.is_prime and p = Nat.2 * h + Nat.1 {
        positive_residues_unique(p)
        positive_residues(p).is_unique
        filter_preserves_unique(positive_residues(p), is_unit_qr_at(p))
        unit_qr_positive_list(p).is_unique
        half_square_residues_unique(p, h)
        half_square_residues(p, h).is_unique
        forall(x: Nat) {
            unit_qr_positive_membership_iff_half_square(p, h, x)
            unit_qr_positive_list(p).contains(x) = half_square_residues(p, h).contains(x)
        }
        unique_same_contains_imp_permutation(
            unit_qr_positive_list(p), half_square_residues(p, h))
        is_permutation(unit_qr_positive_list(p), half_square_residues(p, h))
    }
}

/// The number of unit quadratic residues in `1, ..., p-1` is `h = (p-1)/2`.
theorem unit_quadratic_residue_count(p: Nat, h: Nat) {
    p.is_prime and p = Nat.2 * h + Nat.1 implies unit_qr_positive_list(p).length = h
} by {
    if p.is_prime and p = Nat.2 * h + Nat.1 {
        unit_qr_positive_permutation_half_squares(p, h)
        is_permutation(unit_qr_positive_list(p), half_square_residues(p, h))
        permutation_preserves_length(unit_qr_positive_list(p), half_square_residues(p, h))
        unit_qr_positive_list(p).length = half_square_residues(p, h).length
        map_length(h.range, square_suc_mod(p))
        map(h.range, square_suc_mod(p)).length = h.range.length
        half_square_residues(p, h).length = h.range.length
        length_range(h)
        h.range.length = h
        half_square_residues(p, h).length = h
        unit_qr_positive_list(p).length = h
    }
}

// ---------------------------------------------------------------------------
// The count of the unit quadratic nonresidues.
// ---------------------------------------------------------------------------

/// On coprime targets, unit quadratic nonresidues are the complement of unit
/// quadratic residues.
theorem unit_nonqr_iff_not_unit_qr_coprime(p: Nat, x: Nat) {
    x.coprime(p) implies is_unit_quadratic_nonresidue_mod(x, p) = not is_unit_quadratic_residue_mod(x, p)
} by {
    if x.coprime(p) {
        if is_unit_quadratic_nonresidue_mod(x, p) {
            unit_quadratic_nonresidue_is_nonresidue(x, p)
            is_quadratic_nonresidue_mod(x, p)
            is_quadratic_nonresidue_mod(x, p) = not is_quadratic_residue_mod(x, p)
            not is_quadratic_residue_mod(x, p)
            quadratic_residue_coprime_iff_unit(x, p)
            is_quadratic_residue_mod(x, p) = is_unit_quadratic_residue_mod(x, p)
            not is_unit_quadratic_residue_mod(x, p)
        }
        if not is_unit_quadratic_residue_mod(x, p) {
            quadratic_residue_coprime_iff_unit(x, p)
            is_quadratic_residue_mod(x, p) = is_unit_quadratic_residue_mod(x, p)
            not is_quadratic_residue_mod(x, p)
            is_quadratic_nonresidue_mod(x, p) = not is_quadratic_residue_mod(x, p)
            is_quadratic_nonresidue_mod(x, p)
            is_unit_quadratic_nonresidue_mod(x, p) =
                (x.coprime(p) and is_quadratic_nonresidue_mod(x, p))
            is_unit_quadratic_nonresidue_mod(x, p)
        }
        (is_unit_quadratic_nonresidue_mod(x, p) =
            not is_unit_quadratic_residue_mod(x, p)) = true
    }
}

/// On the positive residues of a prime, unit quadratic nonresidues are the
/// complement of unit quadratic residues.
theorem unit_nonqr_complement_on_positive(p: Nat, x: Nat) {
    p.is_prime and positive_residues(p).contains(x) implies is_unit_quadratic_nonresidue_mod(x, p) = not is_unit_quadratic_residue_mod(x, p)
} by {
    if p.is_prime and positive_residues(p).contains(x) {
        positive_residues_contains_iff(p, x)
        positive_residues(p).contains(x) = (Nat.0 < x and x < p)
        Nat.0 < x
        x < p
        lte_and_lt(Nat.1, x, p)
        Nat.1 <= x
        coprime_below_prime(p, x)
        x.coprime(p)
        unit_nonqr_iff_not_unit_qr_coprime(p, x)
        is_unit_quadratic_nonresidue_mod(x, p) =
            not is_unit_quadratic_residue_mod(x, p)
    }
}

/// The sum over the positive residues splits into the sum over the unit
/// quadratic residues and the sum over the unit quadratic nonresidues.
theorem sum_map_split_unit_qr_nonqr(p: Nat, f: Nat -> Int) {
    p.is_prime implies
    sum(map(positive_residues(p), f)) =
        sum(map(unit_qr_positive_list(p), f)) +
            sum(map(unit_nonqr_positive_list(p), f))
} by {
    if p.is_prime {
        forall(x: Nat) {
            if positive_residues(p).contains(x) {
                filter_equivalent_to_and(positive_residues(p), is_unit_qr_at(p), x)
                unit_qr_positive_list(p).contains(x) =
                    (positive_residues(p).contains(x) and is_unit_qr_at(p, x))
                filter_equivalent_to_and(positive_residues(p), is_unit_nonqr_at(p), x)
                unit_nonqr_positive_list(p).contains(x) =
                    (positive_residues(p).contains(x) and is_unit_nonqr_at(p, x))
                unit_nonqr_complement_on_positive(p, x)
                is_unit_quadratic_nonresidue_mod(x, p) =
                    not is_unit_quadratic_residue_mod(x, p)
                is_unit_nonqr_at(p, x) = not is_unit_qr_at(p, x)
                add_contains_or(unit_qr_positive_list(p), unit_nonqr_positive_list(p), x)
                (unit_qr_positive_list(p) + unit_nonqr_positive_list(p)).contains(x) =
                    unit_qr_positive_list(p).contains(x) or
                        unit_nonqr_positive_list(p).contains(x)
                if is_unit_qr_at(p, x) {
                    is_unit_nonqr_at(p, x) = not is_unit_qr_at(p, x)
                    not is_unit_qr_at(p, x) = false
                    is_unit_nonqr_at(p, x) = false
                    unit_qr_positive_list(p).contains(x) =
                        (positive_residues(p).contains(x) and is_unit_qr_at(p, x))
                    unit_qr_positive_list(p).contains(x)
                    unit_nonqr_positive_list(p).contains(x) =
                        (positive_residues(p).contains(x) and is_unit_nonqr_at(p, x))
                    not unit_nonqr_positive_list(p).contains(x)
                    unit_qr_positive_list(p).contains(x) or
                        unit_nonqr_positive_list(p).contains(x)
                    (unit_qr_positive_list(p) + unit_nonqr_positive_list(p)).contains(x)
                }
                if not is_unit_qr_at(p, x) {
                    is_unit_nonqr_at(p, x) = not is_unit_qr_at(p, x)
                    not is_unit_qr_at(p, x) = true
                    is_unit_nonqr_at(p, x) = true
                    unit_qr_positive_list(p).contains(x) =
                        (positive_residues(p).contains(x) and is_unit_qr_at(p, x))
                    not unit_qr_positive_list(p).contains(x)
                    unit_nonqr_positive_list(p).contains(x) =
                        (positive_residues(p).contains(x) and is_unit_nonqr_at(p, x))
                    unit_nonqr_positive_list(p).contains(x)
                    unit_qr_positive_list(p).contains(x) or
                        unit_nonqr_positive_list(p).contains(x)
                    (unit_qr_positive_list(p) + unit_nonqr_positive_list(p)).contains(x)
                }
                (unit_qr_positive_list(p) + unit_nonqr_positive_list(p)).contains(x) =
                    positive_residues(p).contains(x)
            }
            if not positive_residues(p).contains(x) {
                filter_equivalent_to_and(positive_residues(p), is_unit_qr_at(p), x)
                unit_qr_positive_list(p).contains(x) =
                    (positive_residues(p).contains(x) and is_unit_qr_at(p, x))
                not unit_qr_positive_list(p).contains(x)
                filter_equivalent_to_and(positive_residues(p), is_unit_nonqr_at(p), x)
                unit_nonqr_positive_list(p).contains(x) =
                    (positive_residues(p).contains(x) and is_unit_nonqr_at(p, x))
                not unit_nonqr_positive_list(p).contains(x)
                add_contains_or(unit_qr_positive_list(p), unit_nonqr_positive_list(p), x)
                (unit_qr_positive_list(p) + unit_nonqr_positive_list(p)).contains(x) =
                    unit_qr_positive_list(p).contains(x) or
                        unit_nonqr_positive_list(p).contains(x)
                not (unit_qr_positive_list(p) + unit_nonqr_positive_list(p)).contains(x)
                (unit_qr_positive_list(p) + unit_nonqr_positive_list(p)).contains(x) =
                    positive_residues(p).contains(x)
            }
            (unit_qr_positive_list(p) + unit_nonqr_positive_list(p)).contains(x) =
                positive_residues(p).contains(x)
        }
        forall(x: Nat) {
            if unit_qr_positive_list(p).contains(x) and
                    unit_nonqr_positive_list(p).contains(x) {
                filter_equivalent_to_and(positive_residues(p), is_unit_qr_at(p), x)
                unit_qr_positive_list(p).contains(x) =
                    (positive_residues(p).contains(x) and is_unit_qr_at(p, x))
                positive_residues(p).contains(x)
                is_unit_qr_at(p, x)
                filter_equivalent_to_and(positive_residues(p), is_unit_nonqr_at(p), x)
                unit_nonqr_positive_list(p).contains(x) =
                    (positive_residues(p).contains(x) and is_unit_nonqr_at(p, x))
                is_unit_nonqr_at(p, x)
                unit_nonqr_complement_on_positive(p, x)
                is_unit_quadratic_nonresidue_mod(x, p) =
                    not is_unit_quadratic_residue_mod(x, p)
                is_unit_nonqr_at(p, x) = not is_unit_qr_at(p, x)
                if is_unit_qr_at(p, x) {
                    not is_unit_qr_at(p, x) = false
                    is_unit_nonqr_at(p, x) = false
                    false
                }
                if not is_unit_qr_at(p, x) {
                    false
                }
                false
            }
            not (unit_qr_positive_list(p).contains(x) and
                unit_nonqr_positive_list(p).contains(x))
        }
        filter_preserves_unique(positive_residues(p), is_unit_qr_at(p))
        unit_qr_positive_list(p).is_unique
        filter_preserves_unique(positive_residues(p), is_unit_nonqr_at(p))
        unit_nonqr_positive_list(p).is_unique
        unique_list_sum(unit_qr_positive_list(p), unit_nonqr_positive_list(p))
        (unit_qr_positive_list(p) + unit_nonqr_positive_list(p)).is_unique
        positive_residues_unique(p)
        positive_residues(p).is_unique
        unique_same_contains_map_sum_eq(
            unit_qr_positive_list(p) + unit_nonqr_positive_list(p),
            positive_residues(p), f)
        sum(map(unit_qr_positive_list(p) + unit_nonqr_positive_list(p), f)) =
            sum(map(positive_residues(p), f))
        map_add(unit_qr_positive_list(p), unit_nonqr_positive_list(p), f)
        map(unit_qr_positive_list(p) + unit_nonqr_positive_list(p), f) =
            map(unit_qr_positive_list(p), f) + map(unit_nonqr_positive_list(p), f)
        sum_add(map(unit_qr_positive_list(p), f), map(unit_nonqr_positive_list(p), f))
        sum(map(unit_qr_positive_list(p), f) + map(unit_nonqr_positive_list(p), f)) =
            sum(map(unit_qr_positive_list(p), f)) +
                sum(map(unit_nonqr_positive_list(p), f))
        sum(map(unit_qr_positive_list(p), f)) +
            sum(map(unit_nonqr_positive_list(p), f)) =
            sum(map(positive_residues(p), f))
        sum(map(positive_residues(p), f)) =
            sum(map(unit_qr_positive_list(p), f)) +
                sum(map(unit_nonqr_positive_list(p), f))
    }
}

/// The unit quadratic residues and nonresidues in `1, ..., p-1` together have
/// length `p - 1`.
theorem unit_qr_nonqr_length_pair(p: Nat, h: Nat) {
    p.is_prime and p = Nat.2 * h + Nat.1 implies unit_qr_positive_list(p).length + unit_nonqr_positive_list(p).length =
            positive_residues(p).length
} by {
    if p.is_prime and p = Nat.2 * h + Nat.1 {
        sum_map_split_unit_qr_nonqr(p, int_one_summand)
        sum(map(positive_residues(p), int_one_summand)) =
            sum(map(unit_qr_positive_list(p), int_one_summand)) +
                sum(map(unit_nonqr_positive_list(p), int_one_summand))
        int_sum_const_one(positive_residues(p))
        sum(map(positive_residues(p), int_one_summand)) =
            Int.from_nat(positive_residues(p).length)
        int_sum_const_one(unit_qr_positive_list(p))
        sum(map(unit_qr_positive_list(p), int_one_summand)) =
            Int.from_nat(unit_qr_positive_list(p).length)
        int_sum_const_one(unit_nonqr_positive_list(p))
        sum(map(unit_nonqr_positive_list(p), int_one_summand)) =
            Int.from_nat(unit_nonqr_positive_list(p).length)
        Int.from_nat(positive_residues(p).length) =
            Int.from_nat(unit_qr_positive_list(p).length) +
                Int.from_nat(unit_nonqr_positive_list(p).length)
        add_from_nat(unit_qr_positive_list(p).length, unit_nonqr_positive_list(p).length)
        Int.from_nat(unit_qr_positive_list(p).length) +
            Int.from_nat(unit_nonqr_positive_list(p).length) =
            Int.from_nat(unit_qr_positive_list(p).length + unit_nonqr_positive_list(p).length)
        Int.from_nat(positive_residues(p).length) =
            Int.from_nat(unit_qr_positive_list(p).length + unit_nonqr_positive_list(p).length)
        unit_qr_positive_list(p).length + unit_nonqr_positive_list(p).length =
            positive_residues(p).length
    }
}

/// The number of unit quadratic residues in `1, ..., p-1` is `(p-1)/2`.
theorem unit_quadratic_residue_count_half(p: Nat, h: Nat) {
    p.is_prime and p = Nat.2 * h + Nat.1 implies
        unit_qr_positive_list(p).length = (p - Nat.1).div(Nat.2)
} by {
    if p.is_prime and p = Nat.2 * h + Nat.1 {
        add_one_right(Nat.2 * h)
        Nat.2 * h + Nat.1 = (Nat.2 * h).suc
        add_imp_sub(Nat.2 * h, Nat.1, p)
        p - Nat.1 = Nat.2 * h
        alt_suc_ne_zero(Nat.1)
        Nat.2 != Nat.0
        div_mul(h, Nat.2)
        (h * Nat.2).div(Nat.2) = h
        Nat.2 * h = h * Nat.2
        (Nat.2 * h).div(Nat.2) = h
        (p - Nat.1).div(Nat.2) = h
        unit_quadratic_residue_count(p, h)
        unit_qr_positive_list(p).length = h
        unit_qr_positive_list(p).length = (p - Nat.1).div(Nat.2)
    }
}

/// The number of unit quadratic nonresidues in `1, ..., p-1` is `h`.
theorem unit_quadratic_nonresidue_count(p: Nat, h: Nat) {
    p.is_prime and p = Nat.2 * h + Nat.1 implies unit_nonqr_positive_list(p).length = h
} by {
    if p.is_prime and p = Nat.2 * h + Nat.1 {
        unit_qr_nonqr_length_pair(p, h)
        unit_qr_positive_list(p).length + unit_nonqr_positive_list(p).length =
            positive_residues(p).length
        map_length((p - Nat.1).range, Nat.suc)
        map((p - Nat.1).range, Nat.suc).length = (p - Nat.1).range.length
        positive_residues(p).length = (p - Nat.1).range.length
        length_range(p - Nat.1)
        (p - Nat.1).range.length = p - Nat.1
        positive_residues(p).length = p - Nat.1
        add_one_right(Nat.2 * h)
        Nat.2 * h + Nat.1 = (Nat.2 * h).suc
        add_imp_sub(Nat.2 * h, Nat.1, p)
        p - Nat.1 = Nat.2 * h
        mul_two_left(h)
        Nat.2 * h = h + h
        positive_residues(p).length = h + h
        unit_qr_positive_list(p).length + unit_nonqr_positive_list(p).length = h + h
        unit_quadratic_residue_count(p, h)
        unit_qr_positive_list(p).length = h
        h + unit_nonqr_positive_list(p).length = h + h
        add_cancels_left(h, unit_nonqr_positive_list(p).length, h)
        unit_nonqr_positive_list(p).length = h
    }
}

// ---------------------------------------------------------------------------
// The Legendre symbol is one on unit quadratic residues and minus one on unit
// quadratic nonresidues, pointwise over the positive residues.
// ---------------------------------------------------------------------------

/// The Legendre symbol is one on a unit quadratic residue.
theorem legendre_symbol_one_of_unit_qr(p: Nat, a: Nat) {
    p.is_prime and is_unit_quadratic_residue_mod(a, p)
        implies legendre_symbol(a, p) = Int.1
} by {
    if p.is_prime and is_unit_quadratic_residue_mod(a, p) {
        legendre_symbol_one_of_prime_unit_quadratic_residue(p, a)
        legendre_symbol(a, p) = Int.1
    }
}

/// The Legendre symbol is minus one on a unit quadratic nonresidue.
theorem legendre_symbol_neg_one_of_unit_nonqr(p: Nat, a: Nat) {
    p.is_prime and is_unit_quadratic_nonresidue_mod(a, p)
        implies legendre_symbol(a, p) = -Int.1
} by {
    if p.is_prime and is_unit_quadratic_nonresidue_mod(a, p) {
        legendre_symbol_neg_one_of_prime_unit_nonresidue(p, a)
        legendre_symbol(a, p) = -Int.1
    }
}

// ---------------------------------------------------------------------------
// The product formula: sum_{a=1}^{p-1} (a/p) = 0.
// ---------------------------------------------------------------------------

/// The sum of the Legendre symbols over `1, ..., p-1` vanishes for every odd
/// prime `p = 2h+1`.
theorem legendre_symbol_sum_positive_zero(p: Nat, h: Nat) {
    p.is_prime and p = Nat.2 * h + Nat.1 implies legendre_symbol_sum_positive(p) = Int.0
} by {
    if p.is_prime and p = Nat.2 * h + Nat.1 {
        sum_map_split_unit_qr_nonqr(p, legendre_at(p))
        sum(map(positive_residues(p), legendre_at(p))) =
            sum(map(unit_qr_positive_list(p), legendre_at(p))) +
                sum(map(unit_nonqr_positive_list(p), legendre_at(p)))
        forall(x: Nat) {
            if unit_qr_positive_list(p).contains(x) {
                filter_equivalent_to_and(positive_residues(p), is_unit_qr_at(p), x)
                unit_qr_positive_list(p).contains(x) =
                    (positive_residues(p).contains(x) and is_unit_qr_at(p, x))
                is_unit_qr_at(p, x)
                is_unit_quadratic_residue_mod(x, p)
                legendre_symbol_one_of_unit_qr(p, x)
                legendre_symbol(x, p) = Int.1
                legendre_at(p, x) = legendre_symbol(x, p)
                int_one_summand(x) = Int.1
                legendre_at(p, x) = int_one_summand(x)
            }
        }
        sum_map_of_pointwise(unit_qr_positive_list(p), legendre_at(p), int_one_summand)
        sum(map(unit_qr_positive_list(p), legendre_at(p))) =
            sum(map(unit_qr_positive_list(p), int_one_summand))
        int_sum_const_one(unit_qr_positive_list(p))
        sum(map(unit_qr_positive_list(p), int_one_summand)) =
            Int.from_nat(unit_qr_positive_list(p).length)
        unit_quadratic_residue_count(p, h)
        unit_qr_positive_list(p).length = h
        sum(map(unit_qr_positive_list(p), legendre_at(p))) = Int.from_nat(h)
        forall(x: Nat) {
            if unit_nonqr_positive_list(p).contains(x) {
                filter_equivalent_to_and(positive_residues(p), is_unit_nonqr_at(p), x)
                unit_nonqr_positive_list(p).contains(x) =
                    (positive_residues(p).contains(x) and is_unit_nonqr_at(p, x))
                is_unit_nonqr_at(p, x)
                is_unit_quadratic_nonresidue_mod(x, p)
                legendre_symbol_neg_one_of_unit_nonqr(p, x)
                legendre_symbol(x, p) = -Int.1
                legendre_at(p, x) = legendre_symbol(x, p)
                int_neg_one_summand(x) = -Int.1
                legendre_at(p, x) = int_neg_one_summand(x)
            }
        }
        sum_map_of_pointwise(unit_nonqr_positive_list(p), legendre_at(p), int_neg_one_summand)
        sum(map(unit_nonqr_positive_list(p), legendre_at(p))) =
            sum(map(unit_nonqr_positive_list(p), int_neg_one_summand))
        int_sum_const_neg_one(unit_nonqr_positive_list(p))
        sum(map(unit_nonqr_positive_list(p), int_neg_one_summand)) =
            -(Int.from_nat(unit_nonqr_positive_list(p).length))
        unit_quadratic_nonresidue_count(p, h)
        unit_nonqr_positive_list(p).length = h
        sum(map(unit_nonqr_positive_list(p), legendre_at(p))) = -(Int.from_nat(h))
        legendre_symbol_sum_positive(p) = sum(map(positive_residues(p), legendre_at(p)))
        legendre_symbol_sum_positive(p) =
            sum(map(unit_qr_positive_list(p), legendre_at(p))) +
                sum(map(unit_nonqr_positive_list(p), legendre_at(p)))
        legendre_symbol_sum_positive(p) = Int.from_nat(h) + -(Int.from_nat(h))
        add_neg(Int.from_nat(h))
        Int.from_nat(h) + -(Int.from_nat(h)) = Int.0
        legendre_symbol_sum_positive(p) = Int.0
    }
}

// ---------------------------------------------------------------------------
// The supplements.
// ---------------------------------------------------------------------------

/// The first supplement: `(-1/p) = 1` when `(p-1)/2` is even and `-1`
/// otherwise. This is `prime_pred_legendre_symbol_by_half_parity` from
/// `quadratic_residue_supplements.ac`, restated for the applications.
theorem legendre_symbol_neg_one_by_half_parity(p: Nat, h: Nat) {
    p.is_prime and p = Nat.2 * h + Nat.1 implies legendre_symbol(p - Nat.1, p) =
            if Nat.2.divides(h) { Int.1 } else { -Int.1 }
} by {
    if p.is_prime and p = Nat.2 * h + Nat.1 {
        prime_pred_legendre_symbol_by_half_parity(p, h)
        legendre_symbol(p - Nat.1, p) =
            if Nat.2.divides(h) { Int.1 } else { -Int.1 }
    }
}

/// The second supplement: `(2/p) = 1` in the residue classes `1` and `7`
/// modulo eight and `-1` otherwise, assuming a full unit generator exists.
/// This is `prime_two_legendre_symbol_mod_eight` from
/// `quadratic_residue_supplements.ac`.
theorem legendre_symbol_two_by_mod_eight(p: Nat, h: Nat) {
    p.is_prime and p = Nat.2 * h + Nat.1 and
        (exists(g: Nat) { is_order_double_unit_generator_mod(g, p, h) })
    implies legendre_symbol(Nat.2, p) =
        if p.congr_mod(Nat.1, Nat.8) or p.congr_mod(Nat.7, Nat.8) {
            Int.1
        } else {
            -Int.1
        }
} by {
    if p.is_prime and p = Nat.2 * h + Nat.1 and
            (exists(g: Nat) { is_order_double_unit_generator_mod(g, p, h) }) {
        prime_two_legendre_symbol_mod_eight(p, h)
        legendre_symbol(Nat.2, p) =
            if p.congr_mod(Nat.1, Nat.8) or p.congr_mod(Nat.7, Nat.8) {
                Int.1
            } else {
                -Int.1
            }
    }
}

// ---------------------------------------------------------------------------
// Closure properties.
// ---------------------------------------------------------------------------

/// The product of two quadratic residues modulo `p` is a quadratic residue.
theorem quadratic_residue_product_is_residue(a: Nat, b: Nat, p: Nat) {
    is_quadratic_residue_mod(a, p) and is_quadratic_residue_mod(b, p)
        implies is_quadratic_residue_mod(a * b, p)
} by {
    if is_quadratic_residue_mod(a, p) and is_quadratic_residue_mod(b, p) {
        quadratic_residue_mul(a, b, p)
        is_quadratic_residue_mod(a * b, p)
    }
}

/// The product of two unit quadratic residues modulo `p` is a unit quadratic
/// residue.
theorem unit_quadratic_residue_product_is_residue(a: Nat, b: Nat, p: Nat) {
    is_unit_quadratic_residue_mod(a, p) and is_unit_quadratic_residue_mod(b, p)
        implies is_unit_quadratic_residue_mod(a * b, p)
} by {
    if is_unit_quadratic_residue_mod(a, p) and is_unit_quadratic_residue_mod(b, p) {
        unit_quadratic_residue_mul(a, b, p)
        is_unit_quadratic_residue_mod(a * b, p)
    }
}
