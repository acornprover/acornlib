// Goldbach's conjecture and its weak form.
//
// Goldbach's conjecture — every even natural number at least 4 is a sum of
// two primes — is unproved.  The weak Goldbach conjecture — every odd natural
// number at least 7 is a sum of three primes — was proved by Helfgott (2013).
// This file records finite verifications of both statements for small
// numbers, proves the even cases 4, 6, 8, 10, 12 and 14 of the range
// verification, and states the general statements without proof.

from nat import Nat, lt_suc_right, lt_imp_lte_suc, lt_imp_lt_suc, lte_and_lt,
    lt_not_ref, not_lt_zero, mul_two_left, lt_cancel_mul, mul_one_right,
    mul_zero_right, lt_trans, add_cancels_left, add_zero_right, add_one_left,
    add_imp_sub, divides_sub, add_assoc, add_comm, lt_suc
from number_theory.coprime import nat_divides_one_imp_one
from number_theory.falling_product import nat_two_prime
from number_theory.factorisation import no_proper_divisor_imp_prime

numerals Nat

// ============================================================================
// The predicates of Goldbach's conjecture and its weak form.
// ============================================================================

/// A natural number is a sum of two primes.
define is_sum_of_two_primes(n: Nat) -> Bool {
    exists(p: Nat, q: Nat) { p.is_prime and q.is_prime and p + q = n }
}

/// A natural number is a sum of three primes.
define is_sum_of_three_primes(n: Nat) -> Bool {
    exists(p: Nat, q: Nat, r: Nat) { p.is_prime and q.is_prime and r.is_prime and p + q + r = n }
}

/// Every even natural number at least 4 that is at most `n` is a sum of two primes.
///
/// The upper bound is included so that finite range verifications can be
/// expressed; with no upper bound this is Goldbach's conjecture.
define goldbach_even_upto(n: Nat, m: Nat) -> Bool {
    Nat.4 <= n and n <= m and n.mod(Nat.2) = Nat.0 implies is_sum_of_two_primes(n)
}

/// Every odd natural number at least 7 that is at most `n` is a sum of three primes.
///
/// The upper bound is included so that finite range verifications can be
/// expressed; with no upper bound this is the weak Goldbach theorem.
define weak_goldbach_odd_upto(n: Nat, m: Nat) -> Bool {
    Nat.7 <= n and n <= m and n.mod(Nat.2) = Nat.1 implies is_sum_of_three_primes(n)
}

// ============================================================================
// Primality of the small numbers 2, 3, 5 and 7.
// ============================================================================

/// If `a + c = b` with `c` nonzero, then `a` and `b` differ.
theorem lt_ne(a: Nat, b: Nat, c: Nat) {
    a + c = b and c != Nat.0 implies a != b
} by {
    if a + c = b and c != Nat.0 {
        if a = b {
            a + c = a
            a + Nat.0 = a
            add_cancels_left(a, c, Nat.0)
            c = Nat.0
            false
        }
        a != b
    }
}

/// The successor of a number minus the number is one.
theorem suc_sub_self(a: Nat) {
    a.suc - a = Nat.1
} by {
    add_one_left(a)
    Nat.1 + a = a.suc
    add_imp_sub(Nat.1, a, a.suc)
    a.suc - a = Nat.1
}

/// A divisor of two consecutive numbers divides one, hence equals one.
theorem divides_suc_pair_imp_one(d: Nat, m: Nat) {
    d.divides(m) and d.divides(m.suc) implies d = Nat.1
} by {
    if d.divides(m) and d.divides(m.suc) {
        divides_sub(m.suc, m, d)
        suc_sub_self(m)
        m.suc - m = Nat.1
        d.divides(m.suc - m)
        d.divides(Nat.1)
        nat_divides_one_imp_one(d)
        d = Nat.1
    }
}

/// `1 < 3`.
theorem one_lt_three {
    Nat.1 < Nat.3
} by {
    Nat.1 < Nat.2
    lt_imp_lt_suc(Nat.1, Nat.2)
    Nat.1 < Nat.3
}

/// `1 < 4`.
theorem one_lt_four {
    Nat.1 < Nat.4
} by {
    one_lt_three
    Nat.1 < Nat.3
    lt_imp_lt_suc(Nat.1, Nat.3)
    Nat.1 < Nat.4
}

/// `1 < 5`.
theorem one_lt_five {
    Nat.1 < Nat.5
} by {
    one_lt_four
    Nat.1 < Nat.4
    lt_imp_lt_suc(Nat.1, Nat.4)
    Nat.1 < Nat.5
}

/// `1 < 6`.
theorem one_lt_six {
    Nat.1 < Nat.6
} by {
    one_lt_five
    Nat.1 < Nat.5
    lt_imp_lt_suc(Nat.1, Nat.5)
    Nat.1 < Nat.6
}

/// `1 < 7`.
theorem one_lt_seven {
    Nat.1 < Nat.7
} by {
    one_lt_six
    Nat.1 < Nat.6
    lt_imp_lt_suc(Nat.1, Nat.6)
    Nat.1 < Nat.7
}

/// `3 < 5`.
theorem three_lt_five {
    Nat.3 < Nat.5
} by {
    lt_suc(Nat.3)
    Nat.3 < Nat.4
    lt_suc(Nat.4)
    Nat.4 < Nat.5
    lt_trans(Nat.3, Nat.4, Nat.5)
    Nat.3 < Nat.5
}

/// `5 < 7`.
theorem five_lt_seven {
    Nat.5 < Nat.7
} by {
    lt_suc(Nat.5)
    Nat.5 < Nat.6
    lt_suc(Nat.6)
    Nat.6 < Nat.7
    lt_trans(Nat.5, Nat.6, Nat.7)
    Nat.5 < Nat.7
}

/// `4 < 7`.
theorem four_lt_seven {
    Nat.4 < Nat.7
} by {
    lt_suc(Nat.4)
    Nat.4 < Nat.5
    five_lt_seven
    Nat.5 < Nat.7
    lt_trans(Nat.4, Nat.5, Nat.7)
    Nat.4 < Nat.7
}

/// `5 < 8`.
theorem five_lt_eight {
    Nat.5 < Nat.8
} by {
    five_lt_seven
    Nat.5 < Nat.7
    lt_imp_lt_suc(Nat.5, Nat.7)
    Nat.5 < Nat.8
}

/// `7 < 10`.
theorem seven_lt_ten {
    Nat.7 < Nat.10
} by {
    lt_suc(Nat.7)
    Nat.7 < Nat.8
    lt_imp_lt_suc(Nat.7, Nat.8)
    Nat.7 < Nat.9
    lt_imp_lt_suc(Nat.7, Nat.9)
    Nat.7 < Nat.10
}

/// `7 < 12`.
theorem seven_lt_twelve {
    Nat.7 < Nat.12
} by {
    seven_lt_ten
    Nat.7 < Nat.10
    lt_imp_lt_suc(Nat.7, Nat.10)
    Nat.7 < Nat.11
    (Nat.7 < Nat.11.suc)
    (Nat.11.suc = Nat.12)
    Nat.7 < Nat.12
}

/// `2 != 1`.
theorem two_ne_one {
    Nat.2 != Nat.1
} by {
    lt_ne(Nat.1, Nat.2, Nat.1)
    Nat.1 + Nat.1 = Nat.2
    Nat.1 != Nat.0
    Nat.1 != Nat.2
    Nat.2 != Nat.1
}

/// `3 != 1`.
theorem three_ne_one {
    Nat.3 != Nat.1
} by {
    lt_ne(Nat.1, Nat.3, Nat.2)
    Nat.1 + Nat.2 = Nat.3
    Nat.2 != Nat.0
}

/// A number strictly between `d` and `2d` is not a multiple of `d`.
///
/// If `d` divided `n < 2d` then the quotient would be positive and below 2,
/// hence 1, forcing `d = n` and contradicting `d < n`.
theorem not_divides_below_twice(d: Nat, n: Nat) {
    Nat.1 < d and d < n and n < d + d implies not d.divides(n)
} by {
    if Nat.1 < d and d < n and n < d + d {
        if d.divides(n) {
            (d.divides(n) = exists(c: Nat) { d * c = n })
            let (c: Nat) satisfy {
                d * c = n
            }
            Nat.0 < Nat.1
            lt_trans(Nat.0, Nat.1, d)
            Nat.0 < d
            lt_trans(Nat.0, d, n)
            Nat.0 < n
            (d * c < d + d)
            mul_two_left(d)
            (Nat.2 * d = d + d)
            (d * c < Nat.2 * d)
            d != Nat.0
            lt_cancel_mul(d, c, Nat.2)
            c < Nat.2
            lt_suc_right(c, Nat.1)
            c = Nat.1 or c < Nat.1
            if c < Nat.1 {
                lt_suc_right(c, Nat.0)
                c = Nat.0 or c < Nat.0
                if c < Nat.0 {
                    not_lt_zero(c)
                    false
                } else {
                    c = Nat.0
                    mul_zero_right(d)
                    (d * Nat.0 = Nat.0)
                    (d * c = Nat.0)
                    (Nat.0 = n)
                    Nat.0 < n
                    (Nat.0 < Nat.0)
                    lt_not_ref(Nat.0)
                    false
                }
            } else {
                c = Nat.1
                mul_one_right(d)
                (d * Nat.1 = d)
                (d = n)
                d < n
                (d < d)
                lt_not_ref(d)
                false
            }
        }
        not d.divides(n)
    }
}

/// Two does not divide three.
theorem not_two_divides_three {
    not Nat.2.divides(Nat.3)
} by {
    Nat.1 < Nat.2
    lt_suc(Nat.2)
    Nat.2 < Nat.3
    lt_suc(Nat.3)
    Nat.3 < Nat.4
    (Nat.3 < Nat.2 + Nat.2)
    Nat.1 < Nat.2 and Nat.2 < Nat.3 and Nat.3 < Nat.2 + Nat.2
    not_divides_below_twice(Nat.2, Nat.3)
    not Nat.2.divides(Nat.3)
}

/// Two does not divide five.
theorem not_two_divides_five {
    not Nat.2.divides(Nat.5)
} by {
    if Nat.2.divides(Nat.5) {
        Nat.2 * Nat.2 = Nat.4
        exists(c: Nat) { Nat.2 * c = Nat.4 }
        Nat.2.divides(Nat.4)
        divides_suc_pair_imp_one(Nat.2, Nat.4)
        Nat.2 = Nat.1
        two_ne_one
        false
    }
}

/// Three does not divide five.
theorem not_three_divides_five {
    not Nat.3.divides(Nat.5)
} by {
    one_lt_three
    Nat.1 < Nat.3
    three_lt_five
    Nat.3 < Nat.5
    lt_suc(Nat.5)
    Nat.5 < Nat.6
    (Nat.5 < Nat.3 + Nat.3)
    Nat.1 < Nat.3 and Nat.3 < Nat.5 and Nat.5 < Nat.3 + Nat.3
    not_divides_below_twice(Nat.3, Nat.5)
    not Nat.3.divides(Nat.5)
}

/// Four does not divide five.
theorem not_four_divides_five {
    not Nat.4.divides(Nat.5)
} by {
    one_lt_four
    Nat.1 < Nat.4
    lt_suc(Nat.4)
    Nat.4 < Nat.5
    five_lt_eight
    Nat.5 < Nat.8
    (Nat.5 < Nat.4 + Nat.4)
    Nat.1 < Nat.4 and Nat.4 < Nat.5 and Nat.5 < Nat.4 + Nat.4
    not_divides_below_twice(Nat.4, Nat.5)
    not Nat.4.divides(Nat.5)
}

/// Two does not divide seven.
theorem not_two_divides_seven {
    not Nat.2.divides(Nat.7)
} by {
    if Nat.2.divides(Nat.7) {
        Nat.2 * Nat.3 = Nat.6
        exists(c: Nat) { Nat.2 * c = Nat.6 }
        Nat.2.divides(Nat.6)
        divides_suc_pair_imp_one(Nat.2, Nat.6)
        Nat.2 = Nat.1
        two_ne_one
        false
    }
}

/// Three does not divide seven.
theorem not_three_divides_seven {
    not Nat.3.divides(Nat.7)
} by {
    if Nat.3.divides(Nat.7) {
        Nat.3 * Nat.2 = Nat.6
        exists(c: Nat) { Nat.3 * c = Nat.6 }
        Nat.3.divides(Nat.6)
        divides_suc_pair_imp_one(Nat.3, Nat.6)
        Nat.3 = Nat.1
        three_ne_one
        false
    }
}

/// Four does not divide seven.
theorem not_four_divides_seven {
    not Nat.4.divides(Nat.7)
} by {
    one_lt_four
    Nat.1 < Nat.4
    four_lt_seven
    Nat.4 < Nat.7
    lt_suc(Nat.7)
    Nat.7 < Nat.8
    (Nat.7 < Nat.4 + Nat.4)
    Nat.1 < Nat.4 and Nat.4 < Nat.7 and Nat.7 < Nat.4 + Nat.4
    not_divides_below_twice(Nat.4, Nat.7)
    not Nat.4.divides(Nat.7)
}

/// Five does not divide seven.
theorem not_five_divides_seven {
    not Nat.5.divides(Nat.7)
} by {
    one_lt_five
    Nat.1 < Nat.5
    five_lt_seven
    Nat.5 < Nat.7
    seven_lt_ten
    Nat.7 < Nat.10
    (Nat.7 < Nat.5 + Nat.5)
    Nat.1 < Nat.5 and Nat.5 < Nat.7 and Nat.7 < Nat.5 + Nat.5
    not_divides_below_twice(Nat.5, Nat.7)
    not Nat.5.divides(Nat.7)
}

/// Six does not divide seven.
theorem not_six_divides_seven {
    not Nat.6.divides(Nat.7)
} by {
    one_lt_six
    Nat.1 < Nat.6
    lt_suc(Nat.6)
    Nat.6 < Nat.7
    seven_lt_twelve
    Nat.7 < Nat.12
    (Nat.7 < Nat.6 + Nat.6)
    Nat.1 < Nat.6 and Nat.6 < Nat.7 and Nat.7 < Nat.6 + Nat.6
    not_divides_below_twice(Nat.6, Nat.7)
    not Nat.6.divides(Nat.7)
}

/// `1 < k` and `k < 3` forces `k = 2`.
theorem k_range_min(k: Nat) {
    Nat.1 < k and k < Nat.3 implies k = Nat.2
} by {
    if Nat.1 < k and k < Nat.3 {
        lt_suc_right(k, Nat.2)
        k = Nat.2 or k < Nat.2
        if k < Nat.2 {
            false
        } else {
            k = Nat.2
            k = Nat.2
        }
    }
}

/// `1 < k` and `k < 4` forces `k` into `{2, 3}`.
theorem k_range_two_three(k: Nat) {
    Nat.1 < k and k < Nat.4 implies (k = Nat.2 or k = Nat.3)
} by {
    if Nat.1 < k and k < Nat.4 {
        lt_suc_right(k, Nat.3)
        k = Nat.3 or k < Nat.3
        if k < Nat.3 {
            k_range_min(k)
            k = Nat.2
            k = Nat.2 or k = Nat.3
        } else {
            k = Nat.3
            k = Nat.2 or k = Nat.3
        }
    }
}

/// `1 < k` and `k < 5` forces `k` into `{2, 3, 4}`.
theorem k_range_two_four(k: Nat) {
    Nat.1 < k and k < Nat.5 implies (k = Nat.2 or k = Nat.3 or k = Nat.4)
} by {
    if Nat.1 < k and k < Nat.5 {
        lt_suc_right(k, Nat.4)
        k = Nat.4 or k < Nat.4
        if k < Nat.4 {
            k_range_two_three(k)
            k = Nat.2 or k = Nat.3 or k = Nat.4
        } else {
            k = Nat.4
            k = Nat.2 or k = Nat.3 or k = Nat.4
        }
    }
}

/// `1 < k` and `k < 6` forces `k` into `{2, 3, 4, 5}`.
theorem k_range_two_five(k: Nat) {
    Nat.1 < k and k < Nat.6 implies (k = Nat.2 or k = Nat.3 or k = Nat.4 or k = Nat.5)
} by {
    if Nat.1 < k and k < Nat.6 {
        lt_suc_right(k, Nat.5)
        k = Nat.5 or k < Nat.5
        if k < Nat.5 {
            k_range_two_four(k)
            k = Nat.2 or k = Nat.3 or k = Nat.4 or k = Nat.5
        } else {
            k = Nat.5
            k = Nat.2 or k = Nat.3 or k = Nat.4 or k = Nat.5
        }
    }
}

/// `1 < k` and `k < 7` forces `k` into `{2, 3, 4, 5, 6}`.
theorem k_range_two_six(k: Nat) {
    Nat.1 < k and k < Nat.7 implies (k = Nat.2 or k = Nat.3 or k = Nat.4 or k = Nat.5 or k = Nat.6)
} by {
    if Nat.1 < k and k < Nat.7 {
        lt_suc_right(k, Nat.6)
        k = Nat.6 or k < Nat.6
        if k < Nat.6 {
            k_range_two_five(k)
            k = Nat.2 or k = Nat.3 or k = Nat.4 or k = Nat.5 or k = Nat.6
        } else {
            k = Nat.6
            k = Nat.2 or k = Nat.3 or k = Nat.4 or k = Nat.5 or k = Nat.6
        }
    }
}

/// No number strictly between 1 and 3 divides 3.
theorem three_no_proper_divisor(k: Nat) {
    Nat.1 < k and k < Nat.3 implies not k.divides(Nat.3)
} by {
    if Nat.1 < k and k < Nat.3 {
        k_range_min(k)
        k = Nat.2
        if k = Nat.2 {
            not_two_divides_three
            not Nat.2.divides(Nat.3)
            not k.divides(Nat.3)
        } else {
            false
        }
    }
}

/// No number strictly between 1 and 5 divides 5 (by cases).
theorem five_not_divides_by_case(k: Nat) {
    Nat.1 < k and k < Nat.5 and (k = Nat.2 or k = Nat.3 or k = Nat.4)
        implies not k.divides(Nat.5)
} by {
    if Nat.1 < k and k < Nat.5 and (k = Nat.2 or k = Nat.3 or k = Nat.4) {
        if k = Nat.2 {
            not_two_divides_five
            not Nat.2.divides(Nat.5)
            not k.divides(Nat.5)
        } else {
            (k = Nat.3 or k = Nat.4)
            if k = Nat.3 {
                not_three_divides_five
                not Nat.3.divides(Nat.5)
                not k.divides(Nat.5)
            } else {
                k = Nat.4
                not_four_divides_five
                not Nat.4.divides(Nat.5)
                not k.divides(Nat.5)
            }
        }
    }
}

/// No number strictly between 1 and 5 divides 5.
theorem five_no_proper_divisor(k: Nat) {
    Nat.1 < k and k < Nat.5 implies not k.divides(Nat.5)
} by {
    if Nat.1 < k and k < Nat.5 {
        k_range_two_four(k)
        k = Nat.2 or k = Nat.3 or k = Nat.4
        five_not_divides_by_case(k)
        not k.divides(Nat.5)
    }
}

/// No number strictly between 1 and 7 divides 7 (by cases).
theorem seven_not_divides_by_case(k: Nat) {
    Nat.1 < k and k < Nat.7 and (k = Nat.2 or k = Nat.3 or k = Nat.4 or k = Nat.5 or k = Nat.6)
        implies not k.divides(Nat.7)
} by {
    if Nat.1 < k and k < Nat.7 and (k = Nat.2 or k = Nat.3 or k = Nat.4 or k = Nat.5 or k = Nat.6) {
        if k = Nat.2 {
            not_two_divides_seven
            not Nat.2.divides(Nat.7)
            not k.divides(Nat.7)
        } else {
            (k = Nat.3 or k = Nat.4 or k = Nat.5 or k = Nat.6)
            if k = Nat.3 {
                not_three_divides_seven
                not Nat.3.divides(Nat.7)
                not k.divides(Nat.7)
            } else {
                (k = Nat.4 or k = Nat.5 or k = Nat.6)
                if k = Nat.4 {
                    not_four_divides_seven
                    not Nat.4.divides(Nat.7)
                    not k.divides(Nat.7)
                } else {
                    (k = Nat.5 or k = Nat.6)
                    if k = Nat.5 {
                        not_five_divides_seven
                        not Nat.5.divides(Nat.7)
                        not k.divides(Nat.7)
                    } else {
                        k = Nat.6
                        not_six_divides_seven
                        not Nat.6.divides(Nat.7)
                        not k.divides(Nat.7)
                    }
                }
            }
        }
    }
}

/// No number strictly between 1 and 7 divides 7.
theorem seven_no_proper_divisor(k: Nat) {
    Nat.1 < k and k < Nat.7 implies not k.divides(Nat.7)
} by {
    if Nat.1 < k and k < Nat.7 {
        k_range_two_six(k)
        k = Nat.2 or k = Nat.3 or k = Nat.4 or k = Nat.5 or k = Nat.6
        seven_not_divides_by_case(k)
        not k.divides(Nat.7)
    }
}

/// Three is prime.
theorem three_is_prime {
    Nat.3.is_prime
} by {
    one_lt_three
    Nat.1 < Nat.3
    forall(k: Nat) {
        three_no_proper_divisor(k)
    }
    no_proper_divisor_imp_prime(Nat.3)
}

/// Five is prime.
theorem five_is_prime {
    Nat.5.is_prime
} by {
    one_lt_five
    Nat.1 < Nat.5
    forall(k: Nat) {
        five_no_proper_divisor(k)
    }
    no_proper_divisor_imp_prime(Nat.5)
}

/// Seven is prime.
theorem seven_is_prime {
    Nat.7.is_prime
} by {
    one_lt_seven
    Nat.1 < Nat.7
    forall(k: Nat) {
        seven_no_proper_divisor(k)
    }
    no_proper_divisor_imp_prime(Nat.7)
}

// ============================================================================
// Small additions used by the Goldbach decompositions.
// ============================================================================

/// `3 + 3 = 6`.
theorem three_add_three {
    Nat.3 + Nat.3 = Nat.6
} by {
    mul_two_left(Nat.3)
    Nat.2 * Nat.3 = Nat.3 + Nat.3
    Nat.2 * Nat.3 = Nat.6
    (Nat.3 + Nat.3 = Nat.2 * Nat.3)
    Nat.3 + Nat.3 = Nat.6
}

/// `3 + 4 = 7`.
theorem three_add_four {
    Nat.3 + Nat.4 = Nat.7
} by {
    Nat.4 = Nat.3.suc
    Nat.3 + Nat.4 = (Nat.3 + Nat.3).suc
    three_add_three
    Nat.3 + Nat.3 = Nat.6
    Nat.7 = Nat.6.suc
}

/// `3 + 5 = 8`.
theorem three_add_five {
    Nat.3 + Nat.5 = Nat.8
} by {
    Nat.5 = Nat.4.suc
    Nat.3 + Nat.5 = (Nat.3 + Nat.4).suc
    three_add_four
    Nat.3 + Nat.4 = Nat.7
    Nat.8 = Nat.7.suc
}

/// `3 + 6 = 9`.
theorem three_add_six {
    Nat.3 + Nat.6 = Nat.9
} by {
    Nat.6 = Nat.5.suc
    Nat.3 + Nat.6 = (Nat.3 + Nat.5).suc
    three_add_five
    Nat.3 + Nat.5 = Nat.8
    Nat.9 = Nat.8.suc
}

/// `3 + 7 = 10`.
theorem three_add_seven {
    Nat.3 + Nat.7 = Nat.10
} by {
    Nat.7 = Nat.6.suc
    Nat.3 + Nat.7 = (Nat.3 + Nat.6).suc
    three_add_six
    Nat.3 + Nat.6 = Nat.9
    Nat.10 = Nat.9.suc
}

/// `5 + 5 = 10`.
theorem five_add_five {
    Nat.5 + Nat.5 = Nat.10
} by {
    mul_two_left(Nat.5)
    Nat.2 * Nat.5 = Nat.5 + Nat.5
    Nat.2 * Nat.5 = Nat.10
    (Nat.5 + Nat.5 = Nat.2 * Nat.5)
    Nat.5 + Nat.5 = Nat.10
}

/// `5 + 6 = 11`.
theorem five_add_six {
    Nat.5 + Nat.6 = Nat.11
} by {
    Nat.6 = Nat.5.suc
    Nat.5 + Nat.6 = (Nat.5 + Nat.5).suc
    five_add_five
    Nat.5 + Nat.5 = Nat.10
    Nat.11 = Nat.10.suc
}

/// `5 + 7 = 12`.
theorem five_add_seven {
    Nat.5 + Nat.7 = Nat.12
} by {
    Nat.7 = Nat.6.suc
    Nat.5 + Nat.7 = (Nat.5 + Nat.6).suc
    five_add_six
    Nat.5 + Nat.6 = Nat.11
    Nat.12 = Nat.11.suc
}

/// `3 + 3 + 3 = 9`.
theorem three_add_three_add_three {
    Nat.3 + Nat.3 + Nat.3 = Nat.9
} by {
    three_add_three
    Nat.3 + Nat.3 = Nat.6
    (Nat.3 + Nat.3 + Nat.3 = Nat.6 + Nat.3)
    add_comm(Nat.6, Nat.3)
    (Nat.6 + Nat.3 = Nat.3 + Nat.6)
    three_add_six
    Nat.3 + Nat.6 = Nat.9
    Nat.3 + Nat.3 + Nat.3 = Nat.9
}

/// `3 + 3 + 5 = 11`.
theorem three_add_three_add_five {
    Nat.3 + Nat.3 + Nat.5 = Nat.11
} by {
    three_add_three
    Nat.3 + Nat.3 = Nat.6
    (Nat.3 + Nat.3 + Nat.5 = Nat.6 + Nat.5)
    add_comm(Nat.6, Nat.5)
    (Nat.6 + Nat.5 = Nat.5 + Nat.6)
    five_add_six
    Nat.5 + Nat.6 = Nat.11
    Nat.3 + Nat.3 + Nat.5 = Nat.11
}

// ============================================================================
// Goldbach's conjecture for small even numbers.
// ============================================================================

/// Four is the sum of two primes, 4 = 2 + 2.
theorem four_is_sum_of_two_primes {
    is_sum_of_two_primes(Nat.4)
} by {
    nat_two_prime
    Nat.2 + Nat.2 = Nat.4
    exists(p: Nat, q: Nat) { p.is_prime and q.is_prime and p + q = Nat.4 }
    (is_sum_of_two_primes(Nat.4) = exists(p: Nat, q: Nat) { p.is_prime and q.is_prime and p + q = Nat.4 })
    is_sum_of_two_primes(Nat.4)
}

/// Six is the sum of two primes, 6 = 3 + 3.
theorem six_is_sum_of_two_primes {
    is_sum_of_two_primes(Nat.6)
} by {
    three_is_prime
    Nat.3 + Nat.3 = Nat.6
    exists(p: Nat, q: Nat) { p.is_prime and q.is_prime and p + q = Nat.6 }
    (is_sum_of_two_primes(Nat.6) = exists(p: Nat, q: Nat) { p.is_prime and q.is_prime and p + q = Nat.6 })
    is_sum_of_two_primes(Nat.6)
}

/// Eight is the sum of two primes, 8 = 3 + 5.
theorem eight_is_sum_of_two_primes {
    is_sum_of_two_primes(Nat.8)
} by {
    three_is_prime
    five_is_prime
    three_add_five
    Nat.3 + Nat.5 = Nat.8
    exists(p: Nat, q: Nat) { p.is_prime and q.is_prime and p + q = Nat.8 }
    (is_sum_of_two_primes(Nat.8) = exists(p: Nat, q: Nat) { p.is_prime and q.is_prime and p + q = Nat.8 })
    is_sum_of_two_primes(Nat.8)
}

/// Ten is the sum of two primes, 10 = 3 + 7.
theorem ten_is_sum_of_two_primes {
    is_sum_of_two_primes(Nat.10)
} by {
    three_is_prime
    seven_is_prime
    three_add_seven
    Nat.3 + Nat.7 = Nat.10
    exists(p: Nat, q: Nat) { p.is_prime and q.is_prime and p + q = Nat.10 }
    (is_sum_of_two_primes(Nat.10) = exists(p: Nat, q: Nat) { p.is_prime and q.is_prime and p + q = Nat.10 })
    is_sum_of_two_primes(Nat.10)
}

/// Twelve is the sum of two primes, 12 = 5 + 7.
theorem twelve_is_sum_of_two_primes {
    is_sum_of_two_primes(Nat.12)
} by {
    five_is_prime
    seven_is_prime
    five_add_seven
    Nat.5 + Nat.7 = Nat.12
    exists(p: Nat, q: Nat) { p.is_prime and q.is_prime and p + q = Nat.12 }
    (is_sum_of_two_primes(Nat.12) = exists(p: Nat, q: Nat) { p.is_prime and q.is_prime and p + q = Nat.12 })
    is_sum_of_two_primes(Nat.12)
}

/// Fourteen is the sum of two primes, 14 = 7 + 7.
theorem fourteen_is_sum_of_two_primes {
    is_sum_of_two_primes(Nat.14)
} by {
    seven_is_prime
    Nat.7 + Nat.7 = Nat.14
    exists(p: Nat, q: Nat) { p.is_prime and q.is_prime and p + q = Nat.14 }
    (is_sum_of_two_primes(Nat.14) = exists(p: Nat, q: Nat) { p.is_prime and q.is_prime and p + q = Nat.14 })
    is_sum_of_two_primes(Nat.14)
}

// ============================================================================
// The weak Goldbach theorem for small odd numbers.
// ============================================================================

/// Seven is the sum of three primes, 7 = 2 + 2 + 3.
theorem seven_is_sum_of_three_primes {
    is_sum_of_three_primes(Nat.7)
} by {
    nat_two_prime
    three_is_prime
    Nat.2 + Nat.2 + Nat.3 = Nat.7
    exists(p: Nat, q: Nat, r: Nat) { p.is_prime and q.is_prime and r.is_prime and p + q + r = Nat.7 }
    (is_sum_of_three_primes(Nat.7) = exists(p: Nat, q: Nat, r: Nat) { p.is_prime and q.is_prime and r.is_prime and p + q + r = Nat.7 })
    is_sum_of_three_primes(Nat.7)
}

/// Nine is the sum of three primes, 9 = 3 + 3 + 3.
theorem nine_is_sum_of_three_primes {
    is_sum_of_three_primes(Nat.9)
} by {
    three_is_prime
    three_add_three_add_three
    Nat.3 + Nat.3 + Nat.3 = Nat.9
    exists(p: Nat, q: Nat, r: Nat) { p.is_prime and q.is_prime and r.is_prime and p + q + r = Nat.9 }
    (is_sum_of_three_primes(Nat.9) = exists(p: Nat, q: Nat, r: Nat) { p.is_prime and q.is_prime and r.is_prime and p + q + r = Nat.9 })
    is_sum_of_three_primes(Nat.9)
}

/// Eleven is the sum of three primes, 11 = 3 + 3 + 5.
theorem eleven_is_sum_of_three_primes {
    is_sum_of_three_primes(Nat.11)
} by {
    three_is_prime
    five_is_prime
    three_add_three_add_five
    Nat.3 + Nat.3 + Nat.5 = Nat.11
    exists(p: Nat, q: Nat, r: Nat) { p.is_prime and q.is_prime and r.is_prime and p + q + r = Nat.11 }
    (is_sum_of_three_primes(Nat.11) = exists(p: Nat, q: Nat, r: Nat) { p.is_prime and q.is_prime and r.is_prime and p + q + r = Nat.11 })
    is_sum_of_three_primes(Nat.11)
}

// ============================================================================
// The general statements.
// ============================================================================

// Goldbach's conjecture (unproved):
//
//     theorem goldbach_conjecture {
//         forall(n: Nat) { goldbach_even_upto(n, n) }
//     }
//
// The weak Goldbach theorem, proved by Helfgott (2013):
//
//     theorem weak_goldbach_theorem {
//         forall(n: Nat) { weak_goldbach_odd_upto(n, n) }
//     }
//
// Both are deep statements and are stated here without proof.  The finite
// verification for the even range 4..30 uses the decompositions
//
//   4 = 2+2    6 = 3+3    8 = 3+5    10 = 3+7   12 = 5+7   14 = 7+7
//   16 = 5+11  18 = 7+11  20 = 7+13  22 = 11+11 24 = 11+13 26 = 13+13
//   28 = 11+17 30 = 13+17
//
// of which 4, 6, 8, 10, 12 and 14 are proved above; the remaining cases need
// primality of 11, 13 and 17 and are left as a finite verification.
