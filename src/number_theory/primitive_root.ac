from number_theory.quadratic_residue import Nat, is_quadratic_residue_mod,
    is_unit_quadratic_residue_mod, quadratic_residue_coprime_is_unit,
    unit_quadratic_residue_is_residue,
    euler_criterion_unit_quadratic_residue_forward
from number_theory.multiplicative_order import is_multiplicative_order_mod,
    multiplicative_order_divides_exponent, multiplicative_order_mod,
    multiplicative_order_mod_is_order,
    powers_below_multiplicative_order_mod_congr_imp_eq
from number_theory.congruence import congr_mod_refl, congr_mod_symm, congr_mod_trans,
    congr_mod_mul, congr_mod_pow, mod_lt
from number_theory.totient import coprime_residues, coprime_residues_contains_iff,
    coprime_residues_length, coprime_residues_unique
from number_theory.coprime import coprime_mod_imp, coprime_pow_right
from list import List, map, map_contains, map_length, length_range, range_contains_iff_lt,
    range_is_unique, cons_unique_of_tail_unique_not_contains,
    unique_is_smallest_containing_list
from data.finite.finite_fiber_partition import locally_injective_map_is_unique
from nat import exp_add, exp_mul
from nat import divides_cancel_right
numerals Nat

/// True when `a` is represented by a power of `g` modulo `n`.
define is_power_of_mod(a: Nat, g: Nat, n: Nat) -> Bool {
    exists(k: Nat) { a.congr_mod(g.pow(k), n) }
}

/// The powers of `g` cover every reduced residue class modulo `n`.
define powers_cover_units_mod(g: Nat, n: Nat) -> Bool {
    forall(a: Nat) { a.coprime(n) implies is_power_of_mod(a, g, n) }
}

/// The least nonnegative residue of a power of `g` modulo `n`.
define power_mod_fn(g: Nat, n: Nat, k: Nat) -> Nat {
    g.pow(k).mod(n)
}

/// The reduced residues represented by powers below Euler's totient.
define power_mod_residues(g: Nat, n: Nat) -> List[Nat] {
    map(n.totient.range, power_mod_fn(g, n))
}

/// Two unique natural-number lists of equal length have the same membership
/// when the first is contained in the second.
theorem unique_nat_lists_same_length_containment_contains_iff(
    a: List[Nat], b: List[Nat]
) {
    a.is_unique and b.is_unique and a.length = b.length and
        (forall(x: Nat) { a.contains(x) implies b.contains(x) })
        implies forall(x: Nat) { a.contains(x) = b.contains(x) }
} by {
    if a.is_unique and b.is_unique and a.length = b.length and
        (forall(x: Nat) { a.contains(x) implies b.contains(x) }) {
        forall(x: Nat) {
            if b.contains(x) {
                if not a.contains(x) {
                    cons_unique_of_tail_unique_not_contains(x, a)
                    List.cons(x, a).is_unique
                    forall(y: Nat) {
                        if List.cons(x, a).contains(y) {
                            if x = y {
                                b.contains(y)
                            } else {
                                a.contains(y)
                                b.contains(y)
                            }
                        }
                    }
                    unique_is_smallest_containing_list(List.cons(x, a), b)
                    List.cons(x, a).unique.length <= b.length
                    List.cons(x, a).unique = List.cons(x, a)
                    List.cons(x, a).length <= b.length
                    List.cons(x, a).length = a.length.suc
                    a.length.suc <= a.length
                    false
                }
                a.contains(x)
            }
            if a.contains(x) {
                b.contains(x)
            }
            a.contains(x) = b.contains(x)
        }
    }
}

/// Powers below the full selected order give distinct least residues.
theorem full_order_power_mod_residues_unique(g: Nat, n: Nat) {
    n != Nat.0 and g.coprime(n) and multiplicative_order_mod(g, n) = n.totient
        implies power_mod_residues(g, n).is_unique
} by {
    if n != Nat.0 and g.coprime(n) and multiplicative_order_mod(g, n) = n.totient {
        range_is_unique(n.totient)
        forall(i: Nat, j: Nat) {
            if n.totient.range.contains(i) and n.totient.range.contains(j) and
                power_mod_fn(g, n, i) = power_mod_fn(g, n, j) {
                range_contains_iff_lt(n.totient, i)
                range_contains_iff_lt(n.totient, j)
                i < multiplicative_order_mod(g, n)
                j < multiplicative_order_mod(g, n)
                g.pow(i).mod(n) = g.pow(j).mod(n)
                g.pow(i).congr_mod(g.pow(j), n)
                powers_below_multiplicative_order_mod_congr_imp_eq(g, n, i, j)
                i = j
            }
        }
        locally_injective_map_is_unique[Nat, Nat](
            n.totient.range, power_mod_fn(g, n))
        map(n.totient.range, power_mod_fn(g, n)).is_unique
        power_mod_residues(g, n).is_unique
    }
}

/// Every residue represented by a power below the totient is a reduced residue.
theorem power_mod_residues_contained_in_coprime_residues(g: Nat, n: Nat) {
    n != Nat.0 and g.coprime(n) implies forall(x: Nat) {
        power_mod_residues(g, n).contains(x) implies coprime_residues(n).contains(x)
    }
} by {
    if n != Nat.0 and g.coprime(n) {
        forall(x: Nat) {
            if power_mod_residues(g, n).contains(x) {
                map_contains[Nat, Nat](n.totient.range, power_mod_fn(g, n), x)
                let k: Nat satisfy {
                    n.totient.range.contains(k) and power_mod_fn(g, n, k) = x
                }
                coprime_pow_right(g, n, k)
                g.pow(k).coprime(n)
                coprime_mod_imp(g.pow(k), n)
                g.pow(k).mod(n).coprime(n)
                x.coprime(n)
                mod_lt(g.pow(k), n)
                x < n
                coprime_residues_contains_iff(n, x)
                coprime_residues(n).contains(x)
            }
        }
    }
}

/// Full selected multiplicative order makes the powers of `g` cover every
/// reduced residue class.
theorem full_multiplicative_order_powers_cover_units_mod(g: Nat, n: Nat) {
    n != Nat.0 and g.coprime(n) and multiplicative_order_mod(g, n) = n.totient
        implies powers_cover_units_mod(g, n)
} by {
    if n != Nat.0 and g.coprime(n) and multiplicative_order_mod(g, n) = n.totient {
        full_order_power_mod_residues_unique(g, n)
        coprime_residues_unique(n)
        map_length[Nat, Nat](n.totient.range, power_mod_fn(g, n))
        length_range(n.totient)
        power_mod_residues(g, n).length = n.totient
        coprime_residues_length(n)
        coprime_residues(n).length = n.totient
        power_mod_residues_contained_in_coprime_residues(g, n)
        unique_nat_lists_same_length_containment_contains_iff(
            power_mod_residues(g, n), coprime_residues(n))
        forall(x: Nat) {
            power_mod_residues(g, n).contains(x) = coprime_residues(n).contains(x)
        }
        forall(a: Nat) {
            if a.coprime(n) {
                coprime_mod_imp(a, n)
                a.mod(n).coprime(n)
                mod_lt(a, n)
                a.mod(n) < n
                coprime_residues_contains_iff(n, a.mod(n))
                coprime_residues(n).contains(a.mod(n))
                power_mod_residues(g, n).contains(a.mod(n))
                map_contains[Nat, Nat](n.totient.range, power_mod_fn(g, n), a.mod(n))
                let k: Nat satisfy {
                    n.totient.range.contains(k) and
                        power_mod_fn(g, n, k) = a.mod(n)
                }
                a.congr_mod(g.pow(k), n)
                exists(j: Nat) { a.congr_mod(g.pow(j), n) }
                is_power_of_mod(a, g, n)
            }
        }
        powers_cover_units_mod(g, n)
    }
}

/// A literal power of `g` is represented by a power of `g`.
theorem power_of_mod_pow(g: Nat, k: Nat, n: Nat) {
    is_power_of_mod(g.pow(k), g, n)
} by {
    congr_mod_refl(g.pow(k), n)
    exists(j: Nat) { g.pow(k).congr_mod(g.pow(j), n) }
}

/// Congruent targets have the same power-representation status.
theorem power_of_mod_congr_mod(a: Nat, b: Nat, g: Nat, n: Nat) {
    is_power_of_mod(a, g, n) and a.congr_mod(b, n)
        implies is_power_of_mod(b, g, n)
} by {
    if is_power_of_mod(a, g, n) and a.congr_mod(b, n) {
        let k: Nat satisfy { a.congr_mod(g.pow(k), n) }
        congr_mod_symm(a, g.pow(k), n)
        g.pow(k).congr_mod(a, n)
        congr_mod_trans(g.pow(k), a, b, n)
        g.pow(k).congr_mod(b, n)
        congr_mod_symm(g.pow(k), b, n)
        b.congr_mod(g.pow(k), n)
        exists(j: Nat) { b.congr_mod(g.pow(j), n) }
    }
}

/// Congruent targets have equivalent power-representation status.
theorem power_of_mod_congr_mod_iff(a: Nat, b: Nat, g: Nat, n: Nat) {
    a.congr_mod(b, n) implies is_power_of_mod(a, g, n) = is_power_of_mod(b, g, n)
} by {
    if a.congr_mod(b, n) {
        if is_power_of_mod(a, g, n) {
            power_of_mod_congr_mod(a, b, g, n)
            is_power_of_mod(b, g, n)
        }
        if is_power_of_mod(b, g, n) {
            congr_mod_symm(a, b, n)
            b.congr_mod(a, n)
            power_of_mod_congr_mod(b, a, g, n)
            is_power_of_mod(a, g, n)
        }
        (is_power_of_mod(a, g, n) = is_power_of_mod(b, g, n)) = true
    }
}

/// Products of represented powers are represented powers.
theorem power_of_mod_mul(a: Nat, b: Nat, g: Nat, n: Nat) {
    is_power_of_mod(a, g, n) and is_power_of_mod(b, g, n)
        implies is_power_of_mod(a * b, g, n)
} by {
    if is_power_of_mod(a, g, n) and is_power_of_mod(b, g, n) {
        let i: Nat satisfy { a.congr_mod(g.pow(i), n) }
        let j: Nat satisfy { b.congr_mod(g.pow(j), n) }
        congr_mod_mul(a, b, g.pow(i), g.pow(j), n)
        (a * b).congr_mod(g.pow(i) * g.pow(j), n)
        exp_add(g, i, j)
        g.pow(i + j) = g.pow(i) * g.pow(j)
        (a * b).congr_mod(g.pow(i + j), n)
        exists(k: Nat) { (a * b).congr_mod(g.pow(k), n) }
    }
}

/// A target congruent to a product of represented powers is represented.
theorem power_of_mod_mul_congr(a: Nat, b: Nat, c: Nat, g: Nat, n: Nat) {
    is_power_of_mod(a, g, n) and is_power_of_mod(b, g, n)
        and (a * b).congr_mod(c, n)
        implies is_power_of_mod(c, g, n)
} by {
    if is_power_of_mod(a, g, n) and is_power_of_mod(b, g, n)
        and (a * b).congr_mod(c, n) {
        power_of_mod_mul(a, b, g, n)
        is_power_of_mod(a * b, g, n)
        power_of_mod_congr_mod(a * b, c, g, n)
    }
}

/// Powers of represented powers are represented powers.
theorem power_of_mod_power(a: Nat, g: Nat, n: Nat, m: Nat) {
    is_power_of_mod(a, g, n) implies is_power_of_mod(a.pow(m), g, n)
} by {
    if is_power_of_mod(a, g, n) {
        let k: Nat satisfy { a.congr_mod(g.pow(k), n) }
        congr_mod_pow(a, g.pow(k), n, m)
        a.pow(m).congr_mod(g.pow(k).pow(m), n)
        exp_mul(g, k, m)
        g.pow(k * m) = g.pow(k).pow(m)
        a.pow(m).congr_mod(g.pow(k * m), n)
        exists(j: Nat) { a.pow(m).congr_mod(g.pow(j), n) }
    }
}

/// A target congruent to a power of a represented power is represented.
theorem power_of_mod_power_congr(a: Nat, b: Nat, g: Nat, n: Nat, m: Nat) {
    is_power_of_mod(a, g, n) and a.pow(m).congr_mod(b, n)
        implies is_power_of_mod(b, g, n)
} by {
    if is_power_of_mod(a, g, n) and a.pow(m).congr_mod(b, n) {
        power_of_mod_power(a, g, n, m)
        is_power_of_mod(a.pow(m), g, n)
        power_of_mod_congr_mod(a.pow(m), b, g, n)
    }
}

/// Multiplying a represented power by a literal power gives a represented
/// power.
theorem power_of_mod_mul_pow(a: Nat, g: Nat, k: Nat, n: Nat) {
    is_power_of_mod(a, g, n) implies is_power_of_mod(a * g.pow(k), g, n)
} by {
    if is_power_of_mod(a, g, n) {
        power_of_mod_pow(g, k, n)
        is_power_of_mod(g.pow(k), g, n)
        power_of_mod_mul(a, g.pow(k), g, n)
    }
}

/// A target congruent to a represented power times a literal power is
/// represented.
theorem power_of_mod_mul_pow_congr(a: Nat, b: Nat, g: Nat, k: Nat, n: Nat) {
    is_power_of_mod(a, g, n) and (a * g.pow(k)).congr_mod(b, n)
        implies is_power_of_mod(b, g, n)
} by {
    if is_power_of_mod(a, g, n) and (a * g.pow(k)).congr_mod(b, n) {
        power_of_mod_mul_pow(a, g, k, n)
        is_power_of_mod(a * g.pow(k), g, n)
        power_of_mod_congr_mod(a * g.pow(k), b, g, n)
        is_power_of_mod(b, g, n)
    }
}

/// Multiplying a literal power by a represented power gives a represented
/// power.
theorem power_of_mod_pow_mul(g: Nat, k: Nat, a: Nat, n: Nat) {
    is_power_of_mod(a, g, n) implies is_power_of_mod(g.pow(k) * a, g, n)
} by {
    if is_power_of_mod(a, g, n) {
        power_of_mod_pow(g, k, n)
        is_power_of_mod(g.pow(k), g, n)
        power_of_mod_mul(g.pow(k), a, g, n)
    }
}

/// A target congruent to a literal power times a represented power is
/// represented.
theorem power_of_mod_pow_mul_congr(g: Nat, k: Nat, a: Nat, b: Nat, n: Nat) {
    is_power_of_mod(a, g, n) and (g.pow(k) * a).congr_mod(b, n)
        implies is_power_of_mod(b, g, n)
} by {
    if is_power_of_mod(a, g, n) and (g.pow(k) * a).congr_mod(b, n) {
        power_of_mod_pow_mul(g, k, a, n)
        is_power_of_mod(g.pow(k) * a, g, n)
        power_of_mod_congr_mod(g.pow(k) * a, b, g, n)
        is_power_of_mod(b, g, n)
    }
}

/// A unit-coverage hypothesis supplies an explicit power representative for a unit.
theorem powers_cover_units_mod_apply(g: Nat, n: Nat, a: Nat) {
    powers_cover_units_mod(g, n) and a.coprime(n) implies is_power_of_mod(a, g, n)
} by {
    if powers_cover_units_mod(g, n) and a.coprime(n) {
        powers_cover_units_mod(g, n) =
            forall(b: Nat) { b.coprime(n) implies is_power_of_mod(b, g, n) }
        powers_cover_units_mod(g, n) and a.coprime(n)
        (forall(b: Nat) { not b.coprime(n) or is_power_of_mod(b, g, n) }) = true
        function(b: Nat) { not (b.coprime(n) and not is_power_of_mod(b, g, n)) }(a)
        is_power_of_mod(a, g, n)
    }
}

/// Unit coverage gives a power representative for any target congruent to a
/// unit.
theorem powers_cover_units_mod_congr_target(g: Nat, n: Nat, a: Nat, b: Nat) {
    powers_cover_units_mod(g, n) and a.coprime(n) and a.congr_mod(b, n)
        implies is_power_of_mod(b, g, n)
} by {
    if powers_cover_units_mod(g, n) and a.coprime(n) and a.congr_mod(b, n) {
        powers_cover_units_mod_apply(g, n, a)
        is_power_of_mod(a, g, n)
        power_of_mod_congr_mod(a, b, g, n)
    }
}

/// Unit coverage gives a power representative for products of units.
theorem powers_cover_units_mod_mul(g: Nat, n: Nat, a: Nat, b: Nat) {
    powers_cover_units_mod(g, n) and a.coprime(n) and b.coprime(n)
        implies is_power_of_mod(a * b, g, n)
} by {
    if powers_cover_units_mod(g, n) and a.coprime(n) and b.coprime(n) {
        powers_cover_units_mod_apply(g, n, a)
        is_power_of_mod(a, g, n)
        powers_cover_units_mod_apply(g, n, b)
        is_power_of_mod(b, g, n)
        power_of_mod_mul(a, b, g, n)
        is_power_of_mod(a * b, g, n)
    }
}

/// Unit coverage gives a power representative for any target congruent to a
/// product of units.
theorem powers_cover_units_mod_mul_congr(
    g: Nat, n: Nat, a: Nat, b: Nat, c: Nat
) {
    powers_cover_units_mod(g, n) and a.coprime(n) and b.coprime(n)
        and (a * b).congr_mod(c, n)
        implies is_power_of_mod(c, g, n)
} by {
    if powers_cover_units_mod(g, n) and a.coprime(n) and b.coprime(n)
        and (a * b).congr_mod(c, n) {
        powers_cover_units_mod_mul(g, n, a, b)
        is_power_of_mod(a * b, g, n)
        power_of_mod_congr_mod(a * b, c, g, n)
        is_power_of_mod(c, g, n)
    }
}

/// Unit coverage gives a power representative for powers of units.
theorem powers_cover_units_mod_power(g: Nat, n: Nat, a: Nat, m: Nat) {
    powers_cover_units_mod(g, n) and a.coprime(n)
        implies is_power_of_mod(a.pow(m), g, n)
} by {
    if powers_cover_units_mod(g, n) and a.coprime(n) {
        powers_cover_units_mod_apply(g, n, a)
        is_power_of_mod(a, g, n)
        power_of_mod_power(a, g, n, m)
    }
}

/// Unit coverage gives a power representative for any target congruent to a
/// power of a unit.
theorem powers_cover_units_mod_power_congr(
    g: Nat, n: Nat, a: Nat, b: Nat, m: Nat
) {
    powers_cover_units_mod(g, n) and a.coprime(n) and a.pow(m).congr_mod(b, n)
        implies is_power_of_mod(b, g, n)
} by {
    if powers_cover_units_mod(g, n) and a.coprime(n) and a.pow(m).congr_mod(b, n) {
        powers_cover_units_mod_power(g, n, a, m)
        is_power_of_mod(a.pow(m), g, n)
        power_of_mod_congr_mod(a.pow(m), b, g, n)
        is_power_of_mod(b, g, n)
    }
}

/// Unit coverage gives a power representative after multiplying a unit by a
/// literal power of the generator.
theorem powers_cover_units_mod_mul_pow(g: Nat, n: Nat, a: Nat, k: Nat) {
    powers_cover_units_mod(g, n) and a.coprime(n)
        implies is_power_of_mod(a * g.pow(k), g, n)
} by {
    if powers_cover_units_mod(g, n) and a.coprime(n) {
        powers_cover_units_mod_apply(g, n, a)
        is_power_of_mod(a, g, n)
        power_of_mod_mul_pow(a, g, k, n)
        is_power_of_mod(a * g.pow(k), g, n)
    }
}

/// Unit coverage gives a power representative for any target congruent to a
/// unit times a literal generator power.
theorem powers_cover_units_mod_mul_pow_congr(
    g: Nat, n: Nat, a: Nat, b: Nat, k: Nat
) {
    powers_cover_units_mod(g, n) and a.coprime(n)
        and (a * g.pow(k)).congr_mod(b, n)
        implies is_power_of_mod(b, g, n)
} by {
    if powers_cover_units_mod(g, n) and a.coprime(n)
        and (a * g.pow(k)).congr_mod(b, n) {
        powers_cover_units_mod_mul_pow(g, n, a, k)
        is_power_of_mod(a * g.pow(k), g, n)
        power_of_mod_congr_mod(a * g.pow(k), b, g, n)
        is_power_of_mod(b, g, n)
    }
}

/// Unit coverage gives a power representative after multiplying a literal
/// generator power by a unit.
theorem powers_cover_units_mod_pow_mul(g: Nat, n: Nat, k: Nat, a: Nat) {
    powers_cover_units_mod(g, n) and a.coprime(n)
        implies is_power_of_mod(g.pow(k) * a, g, n)
} by {
    if powers_cover_units_mod(g, n) and a.coprime(n) {
        powers_cover_units_mod_apply(g, n, a)
        is_power_of_mod(a, g, n)
        power_of_mod_pow_mul(g, k, a, n)
        is_power_of_mod(g.pow(k) * a, g, n)
    }
}

/// Unit coverage gives a power representative for any target congruent to a
/// literal generator power times a unit.
theorem powers_cover_units_mod_pow_mul_congr(
    g: Nat, n: Nat, k: Nat, a: Nat, b: Nat
) {
    powers_cover_units_mod(g, n) and a.coprime(n)
        and (g.pow(k) * a).congr_mod(b, n)
        implies is_power_of_mod(b, g, n)
} by {
    if powers_cover_units_mod(g, n) and a.coprime(n)
        and (g.pow(k) * a).congr_mod(b, n) {
        powers_cover_units_mod_pow_mul(g, n, k, a)
        is_power_of_mod(g.pow(k) * a, g, n)
        power_of_mod_congr_mod(g.pow(k) * a, b, g, n)
        is_power_of_mod(b, g, n)
    }
}

/// An order-`2*h` generator interface with explicit unit coverage.
define is_order_double_unit_generator_mod(g: Nat, n: Nat, h: Nat) -> Bool {
    n != Nat.0 and g.coprime(n) and h != Nat.0 and
    is_multiplicative_order_mod(g, n, Nat.2 * h) and powers_cover_units_mod(g, n)
}

/// Full selected order equal to `2*h` supplies the order-double unit-generator
/// interface.
theorem full_order_is_order_double_unit_generator_mod(
    g: Nat, n: Nat, h: Nat
) {
    n != Nat.0 and g.coprime(n) and h != Nat.0 and
        multiplicative_order_mod(g, n) = Nat.2 * h and n.totient = Nat.2 * h
        implies is_order_double_unit_generator_mod(g, n, h)
} by {
    if n != Nat.0 and g.coprime(n) and h != Nat.0 and
        multiplicative_order_mod(g, n) = Nat.2 * h and n.totient = Nat.2 * h {
        multiplicative_order_mod_is_order(g, n)
        is_multiplicative_order_mod(g, n, multiplicative_order_mod(g, n))
        is_multiplicative_order_mod(g, n, Nat.2 * h)
        multiplicative_order_mod(g, n) = n.totient
        full_multiplicative_order_powers_cover_units_mod(g, n)
        powers_cover_units_mod(g, n)
        is_order_double_unit_generator_mod(g, n, h)
    }
}

/// Even powers of any base are quadratic residues, up to congruence of targets.
theorem even_power_quadratic_residue_mod(g: Nat, k: Nat, a: Nat, n: Nat) {
    a.congr_mod(g.pow(k), n) and Nat.2.divides(k) implies is_quadratic_residue_mod(a, n)
} by {
    if a.congr_mod(g.pow(k), n) and Nat.2.divides(k) {
        let q: Nat satisfy { Nat.2 * q = k }
        q * Nat.2 = Nat.2 * q
        q * Nat.2 = k
        exp_mul(g, q, Nat.2)
        g.pow(q * Nat.2) = g.pow(q).pow(Nat.2)
        g.pow(k) = g.pow(q).pow(Nat.2)
        g.pow(q).pow(Nat.2) = g.pow(k)
        congr_mod_symm(a, g.pow(k), n)
        g.pow(k).congr_mod(a, n)
        g.pow(q).pow(Nat.2).congr_mod(a, n)
        exists(x: Nat) { x.pow(Nat.2).congr_mod(a, n) }
    }
}

/// If `2*h` is a multiplicative order for `g`, a half-exponent congruence
/// on one of its powers forces that exponent to be even.
theorem order_double_half_power_imp_even_exponent(p: Nat, h: Nat, g: Nat, k: Nat) {
    h != Nat.0 and is_multiplicative_order_mod(g, p, Nat.2 * h)
        and g.pow(k).pow(h).congr_mod(Nat.1, p)
        implies Nat.2.divides(k)
} by {
    if h != Nat.0 and is_multiplicative_order_mod(g, p, Nat.2 * h)
        and g.pow(k).pow(h).congr_mod(Nat.1, p) {
        exp_mul(g, k, h)
        g.pow(k * h) = g.pow(k).pow(h)
        g.pow(k * h).congr_mod(Nat.1, p)
        multiplicative_order_divides_exponent(g, p, Nat.2 * h, k * h)
        (Nat.2 * h).divides(k * h)
        divides_cancel_right(h, Nat.2, k)
        Nat.2.divides(k)
    }
}

/// Conditional Euler-converse bridge for a represented power of an element of
/// multiplicative order `2*h`.
theorem represented_order_double_euler_converse_quadratic_residue(
    p: Nat, h: Nat, g: Nat, k: Nat, a: Nat
) {
    h != Nat.0 and is_multiplicative_order_mod(g, p, Nat.2 * h)
        and a.congr_mod(g.pow(k), p) and a.pow(h).congr_mod(Nat.1, p)
        implies is_quadratic_residue_mod(a, p)
} by {
    if h != Nat.0 and is_multiplicative_order_mod(g, p, Nat.2 * h)
        and a.congr_mod(g.pow(k), p) and a.pow(h).congr_mod(Nat.1, p) {
        congr_mod_pow(a, g.pow(k), p, h)
        a.pow(h).congr_mod(g.pow(k).pow(h), p)
        congr_mod_symm(a.pow(h), g.pow(k).pow(h), p)
        g.pow(k).pow(h).congr_mod(a.pow(h), p)
        congr_mod_trans(g.pow(k).pow(h), a.pow(h), Nat.1, p)
        g.pow(k).pow(h).congr_mod(Nat.1, p)
        order_double_half_power_imp_even_exponent(p, h, g, k)
        Nat.2.divides(k)
        even_power_quadratic_residue_mod(g, k, a, p)
        is_quadratic_residue_mod(a, p)
    }
}

/// Conditional Euler-converse bridge using the power-representation predicate.
theorem power_represented_order_double_euler_converse_quadratic_residue(
    p: Nat, h: Nat, g: Nat, a: Nat
) {
    h != Nat.0 and is_multiplicative_order_mod(g, p, Nat.2 * h)
        and is_power_of_mod(a, g, p) and a.pow(h).congr_mod(Nat.1, p)
        implies is_quadratic_residue_mod(a, p)
} by {
    if h != Nat.0 and is_multiplicative_order_mod(g, p, Nat.2 * h)
        and is_power_of_mod(a, g, p) and a.pow(h).congr_mod(Nat.1, p) {
        let k: Nat satisfy { a.congr_mod(g.pow(k), p) }
        represented_order_double_euler_converse_quadratic_residue(p, h, g, k, a)
        is_quadratic_residue_mod(a, p)
    }
}

/// The packaged order-`2*h` generator interface gives a power representation
/// for every unit residue class.
theorem order_double_unit_generator_covers_unit(g: Nat, p: Nat, h: Nat, a: Nat) {
    is_order_double_unit_generator_mod(g, p, h) and a.coprime(p)
        implies is_power_of_mod(a, g, p)
} by {
    if is_order_double_unit_generator_mod(g, p, h) and a.coprime(p) {
        is_order_double_unit_generator_mod(g, p, h) =
            (p != Nat.0 and g.coprime(p) and h != Nat.0 and
            is_multiplicative_order_mod(g, p, Nat.2 * h) and powers_cover_units_mod(g, p))
        powers_cover_units_mod(g, p)
        a.coprime(p)
        powers_cover_units_mod(g, p) and a.coprime(p)
        powers_cover_units_mod_apply(g, p, a)
        is_power_of_mod(a, g, p)
    }
}

/// A packaged order-`2*h` generator gives a power representation for any
/// target congruent to a unit residue class.
theorem order_double_unit_generator_covers_unit_congr(
    g: Nat, p: Nat, h: Nat, a: Nat, b: Nat
) {
    is_order_double_unit_generator_mod(g, p, h) and a.coprime(p)
        and a.congr_mod(b, p)
        implies is_power_of_mod(b, g, p)
} by {
    if is_order_double_unit_generator_mod(g, p, h) and a.coprime(p)
        and a.congr_mod(b, p) {
        order_double_unit_generator_covers_unit(g, p, h, a)
        is_power_of_mod(a, g, p)
        power_of_mod_congr_mod(a, b, g, p)
        is_power_of_mod(b, g, p)
    }
}

/// A packaged order-`2*h` generator gives a power representation for products
/// of units.
theorem order_double_unit_generator_covers_unit_mul(
    g: Nat, p: Nat, h: Nat, a: Nat, b: Nat
) {
    is_order_double_unit_generator_mod(g, p, h) and a.coprime(p) and b.coprime(p)
        implies is_power_of_mod(a * b, g, p)
} by {
    if is_order_double_unit_generator_mod(g, p, h) and a.coprime(p) and b.coprime(p) {
        order_double_unit_generator_covers_unit(g, p, h, a)
        is_power_of_mod(a, g, p)
        order_double_unit_generator_covers_unit(g, p, h, b)
        is_power_of_mod(b, g, p)
        power_of_mod_mul(a, b, g, p)
        is_power_of_mod(a * b, g, p)
    }
}

/// A packaged order-`2*h` generator gives a power representation for powers
/// of units.
theorem order_double_unit_generator_covers_unit_power(
    g: Nat, p: Nat, h: Nat, a: Nat, m: Nat
) {
    is_order_double_unit_generator_mod(g, p, h) and a.coprime(p)
        implies is_power_of_mod(a.pow(m), g, p)
} by {
    if is_order_double_unit_generator_mod(g, p, h) and a.coprime(p) {
        order_double_unit_generator_covers_unit(g, p, h, a)
        is_power_of_mod(a, g, p)
        power_of_mod_power(a, g, p, m)
        is_power_of_mod(a.pow(m), g, p)
    }
}

/// A packaged order-`2*h` generator gives a power representation for any
/// target congruent to a power of a unit.
theorem order_double_unit_generator_covers_unit_power_congr(
    g: Nat, p: Nat, h: Nat, a: Nat, b: Nat, m: Nat
) {
    is_order_double_unit_generator_mod(g, p, h) and a.coprime(p)
        and a.pow(m).congr_mod(b, p)
        implies is_power_of_mod(b, g, p)
} by {
    if is_order_double_unit_generator_mod(g, p, h) and a.coprime(p)
        and a.pow(m).congr_mod(b, p) {
        order_double_unit_generator_covers_unit_power(g, p, h, a, m)
        is_power_of_mod(a.pow(m), g, p)
        power_of_mod_congr_mod(a.pow(m), b, g, p)
        is_power_of_mod(b, g, p)
    }
}

/// Multiplying a unit by a literal power of a packaged generator gives a
/// represented power.
theorem order_double_unit_generator_covers_unit_mul_pow(
    g: Nat, p: Nat, h: Nat, a: Nat, k: Nat
) {
    is_order_double_unit_generator_mod(g, p, h) and a.coprime(p)
        implies is_power_of_mod(a * g.pow(k), g, p)
} by {
    if is_order_double_unit_generator_mod(g, p, h) and a.coprime(p) {
        order_double_unit_generator_covers_unit(g, p, h, a)
        is_power_of_mod(a, g, p)
        power_of_mod_mul_pow(a, g, k, p)
        is_power_of_mod(a * g.pow(k), g, p)
    }
}

/// A packaged order-`2*h` generator gives a power representation for any
/// target congruent to a unit times a literal generator power.
theorem order_double_unit_generator_covers_unit_mul_pow_congr(
    g: Nat, p: Nat, h: Nat, a: Nat, k: Nat, b: Nat
) {
    is_order_double_unit_generator_mod(g, p, h) and a.coprime(p)
        and (a * g.pow(k)).congr_mod(b, p)
        implies is_power_of_mod(b, g, p)
} by {
    if is_order_double_unit_generator_mod(g, p, h) and a.coprime(p)
        and (a * g.pow(k)).congr_mod(b, p) {
        order_double_unit_generator_covers_unit_mul_pow(g, p, h, a, k)
        is_power_of_mod(a * g.pow(k), g, p)
        power_of_mod_congr_mod(a * g.pow(k), b, g, p)
        is_power_of_mod(b, g, p)
    }
}

/// Multiplying a literal power of a packaged generator by a unit gives a
/// represented power.
theorem order_double_unit_generator_covers_unit_pow_mul(
    g: Nat, p: Nat, h: Nat, k: Nat, a: Nat
) {
    is_order_double_unit_generator_mod(g, p, h) and a.coprime(p)
        implies is_power_of_mod(g.pow(k) * a, g, p)
} by {
    if is_order_double_unit_generator_mod(g, p, h) and a.coprime(p) {
        order_double_unit_generator_covers_unit(g, p, h, a)
        is_power_of_mod(a, g, p)
        power_of_mod_pow_mul(g, k, a, p)
        is_power_of_mod(g.pow(k) * a, g, p)
    }
}

/// A packaged order-`2*h` generator gives a power representation for any
/// target congruent to a literal generator power times a unit.
theorem order_double_unit_generator_covers_unit_pow_mul_congr(
    g: Nat, p: Nat, h: Nat, k: Nat, a: Nat, b: Nat
) {
    is_order_double_unit_generator_mod(g, p, h) and a.coprime(p)
        and (g.pow(k) * a).congr_mod(b, p)
        implies is_power_of_mod(b, g, p)
} by {
    if is_order_double_unit_generator_mod(g, p, h) and a.coprime(p)
        and (g.pow(k) * a).congr_mod(b, p) {
        order_double_unit_generator_covers_unit_pow_mul(g, p, h, k, a)
        is_power_of_mod(g.pow(k) * a, g, p)
        power_of_mod_congr_mod(g.pow(k) * a, b, g, p)
        is_power_of_mod(b, g, p)
    }
}

/// Conditional Euler-converse bridge from unit coverage by a full-order element.
theorem unit_covered_order_double_euler_converse_quadratic_residue(
    p: Nat, h: Nat, g: Nat, a: Nat
) {
    h != Nat.0 and is_multiplicative_order_mod(g, p, Nat.2 * h)
        and powers_cover_units_mod(g, p)
        and a.coprime(p) and a.pow(h).congr_mod(Nat.1, p)
        implies is_quadratic_residue_mod(a, p)
} by {
    if h != Nat.0 and is_multiplicative_order_mod(g, p, Nat.2 * h)
        and powers_cover_units_mod(g, p)
        and a.coprime(p) and a.pow(h).congr_mod(Nat.1, p) {
        powers_cover_units_mod(g, p) and a.coprime(p)
        powers_cover_units_mod_apply(g, p, a)
        is_power_of_mod(a, g, p)
        power_represented_order_double_euler_converse_quadratic_residue(p, h, g, a)
        is_quadratic_residue_mod(a, p)
    }
}

/// Conditional Euler-converse bridge using the packaged order-`2*h` generator interface.
theorem order_double_unit_generator_euler_converse_quadratic_residue(
    p: Nat, h: Nat, g: Nat, a: Nat
) {
    is_order_double_unit_generator_mod(g, p, h)
        and a.coprime(p) and a.pow(h).congr_mod(Nat.1, p)
        implies is_quadratic_residue_mod(a, p)
} by {
    if is_order_double_unit_generator_mod(g, p, h)
        and a.coprime(p) and a.pow(h).congr_mod(Nat.1, p) {
        is_order_double_unit_generator_mod(g, p, h) =
            (p != Nat.0 and g.coprime(p) and h != Nat.0 and
            is_multiplicative_order_mod(g, p, Nat.2 * h) and powers_cover_units_mod(g, p))
        h != Nat.0
        is_multiplicative_order_mod(g, p, Nat.2 * h)
        order_double_unit_generator_covers_unit(g, p, h, a)
        is_power_of_mod(a, g, p)
        power_represented_order_double_euler_converse_quadratic_residue(p, h, g, a)
        is_quadratic_residue_mod(a, p)
    }
}

/// The packaged order-`2*h` generator Euler-converse returns a unit square
/// representative when the input residue is a unit.
theorem order_double_unit_generator_euler_converse_unit_quadratic_residue(
    p: Nat, h: Nat, g: Nat, a: Nat
) {
    is_order_double_unit_generator_mod(g, p, h)
        and a.coprime(p) and a.pow(h).congr_mod(Nat.1, p)
        implies is_unit_quadratic_residue_mod(a, p)
} by {
    if is_order_double_unit_generator_mod(g, p, h)
        and a.coprime(p) and a.pow(h).congr_mod(Nat.1, p) {
        order_double_unit_generator_euler_converse_quadratic_residue(p, h, g, a)
        is_quadratic_residue_mod(a, p)
        quadratic_residue_coprime_is_unit(a, p)
        is_unit_quadratic_residue_mod(a, p)
    }
}

/// Euler's criterion for units, under an explicit generator of the unit group
/// of an odd prime modulus.
theorem order_double_unit_generator_euler_criterion_unit_iff(
    p: Nat, h: Nat, g: Nat, a: Nat
) {
    p.is_prime and p = Nat.2 * h + Nat.1 and
        is_order_double_unit_generator_mod(g, p, h) and a.coprime(p)
        implies (is_unit_quadratic_residue_mod(a, p) =
            a.pow(h).congr_mod(Nat.1, p))
} by {
    if p.is_prime and p = Nat.2 * h + Nat.1 and
        is_order_double_unit_generator_mod(g, p, h) and a.coprime(p) {
        if is_unit_quadratic_residue_mod(a, p) {
            euler_criterion_unit_quadratic_residue_forward(p, h, a)
            a.pow(h).congr_mod(Nat.1, p)
        }
        if a.pow(h).congr_mod(Nat.1, p) {
            order_double_unit_generator_euler_converse_unit_quadratic_residue(
                p, h, g, a)
            is_unit_quadratic_residue_mod(a, p)
        }
        (is_unit_quadratic_residue_mod(a, p) =
            a.pow(h).congr_mod(Nat.1, p)) = true
    }
}

/// Euler's criterion for coprime targets, under an explicit generator of the
/// unit group of an odd prime modulus.
theorem order_double_unit_generator_euler_criterion_iff(
    p: Nat, h: Nat, g: Nat, a: Nat
) {
    p.is_prime and p = Nat.2 * h + Nat.1 and
        is_order_double_unit_generator_mod(g, p, h) and a.coprime(p)
        implies (is_quadratic_residue_mod(a, p) =
            a.pow(h).congr_mod(Nat.1, p))
} by {
    if p.is_prime and p = Nat.2 * h + Nat.1 and
        is_order_double_unit_generator_mod(g, p, h) and a.coprime(p) {
        order_double_unit_generator_euler_criterion_unit_iff(p, h, g, a)
        if is_quadratic_residue_mod(a, p) {
            quadratic_residue_coprime_is_unit(a, p)
            is_unit_quadratic_residue_mod(a, p)
            a.pow(h).congr_mod(Nat.1, p)
        }
        if a.pow(h).congr_mod(Nat.1, p) {
            is_unit_quadratic_residue_mod(a, p)
            unit_quadratic_residue_is_residue(a, p)
            is_quadratic_residue_mod(a, p)
        }
        (is_quadratic_residue_mod(a, p) =
            a.pow(h).congr_mod(Nat.1, p)) = true
    }
}

/// Euler's half-exponent congruence gives a quadratic residue whenever the
/// odd-prime unit group has an order-`2*h` generator.
theorem existing_order_double_unit_generator_euler_converse_quadratic_residue(
    p: Nat, h: Nat, a: Nat
) {
    p.is_prime and p = Nat.2 * h + Nat.1 and
        (exists(g: Nat) { is_order_double_unit_generator_mod(g, p, h) }) and
        a.coprime(p) and a.pow(h).congr_mod(Nat.1, p)
        implies is_quadratic_residue_mod(a, p)
} by {
    if p.is_prime and p = Nat.2 * h + Nat.1 and
        (exists(g: Nat) { is_order_double_unit_generator_mod(g, p, h) }) and
        a.coprime(p) and a.pow(h).congr_mod(Nat.1, p) {
        let g: Nat satisfy { is_order_double_unit_generator_mod(g, p, h) }
        order_double_unit_generator_euler_converse_quadratic_residue(
            p, h, g, a)
        is_quadratic_residue_mod(a, p)
    }
}

/// Euler's half-exponent congruence gives a unit quadratic residue whenever
/// the odd-prime unit group has an order-`2*h` generator.
theorem existing_order_double_unit_generator_euler_converse_unit_quadratic_residue(
    p: Nat, h: Nat, a: Nat
) {
    p.is_prime and p = Nat.2 * h + Nat.1 and
        (exists(g: Nat) { is_order_double_unit_generator_mod(g, p, h) }) and
        a.coprime(p) and a.pow(h).congr_mod(Nat.1, p)
        implies is_unit_quadratic_residue_mod(a, p)
} by {
    if p.is_prime and p = Nat.2 * h + Nat.1 and
        (exists(g: Nat) { is_order_double_unit_generator_mod(g, p, h) }) and
        a.coprime(p) and a.pow(h).congr_mod(Nat.1, p) {
        existing_order_double_unit_generator_euler_converse_quadratic_residue(
            p, h, a)
        is_quadratic_residue_mod(a, p)
        quadratic_residue_coprime_is_unit(a, p)
        is_unit_quadratic_residue_mod(a, p)
    }
}

/// Euler's criterion for units whenever the odd-prime unit group has an
/// order-`2*h` generator.
theorem existing_order_double_unit_generator_euler_criterion_unit_iff(
    p: Nat, h: Nat, a: Nat
) {
    p.is_prime and p = Nat.2 * h + Nat.1 and
        (exists(g: Nat) { is_order_double_unit_generator_mod(g, p, h) }) and
        a.coprime(p)
        implies (is_unit_quadratic_residue_mod(a, p) =
            a.pow(h).congr_mod(Nat.1, p))
} by {
    if p.is_prime and p = Nat.2 * h + Nat.1 and
        (exists(g: Nat) { is_order_double_unit_generator_mod(g, p, h) }) and
        a.coprime(p) {
        let g: Nat satisfy { is_order_double_unit_generator_mod(g, p, h) }
        order_double_unit_generator_euler_criterion_unit_iff(p, h, g, a)
        is_unit_quadratic_residue_mod(a, p) =
            a.pow(h).congr_mod(Nat.1, p)
    }
}

/// Euler's criterion for coprime classes whenever the odd-prime unit group
/// has an order-`2*h` generator.
theorem existing_order_double_unit_generator_euler_criterion_iff(
    p: Nat, h: Nat, a: Nat
) {
    p.is_prime and p = Nat.2 * h + Nat.1 and
        (exists(g: Nat) { is_order_double_unit_generator_mod(g, p, h) }) and
        a.coprime(p)
        implies (is_quadratic_residue_mod(a, p) =
            a.pow(h).congr_mod(Nat.1, p))
} by {
    if p.is_prime and p = Nat.2 * h + Nat.1 and
        (exists(g: Nat) { is_order_double_unit_generator_mod(g, p, h) }) and
        a.coprime(p) {
        let g: Nat satisfy { is_order_double_unit_generator_mod(g, p, h) }
        order_double_unit_generator_euler_criterion_iff(p, h, g, a)
        is_quadratic_residue_mod(a, p) =
            a.pow(h).congr_mod(Nat.1, p)
    }
}
