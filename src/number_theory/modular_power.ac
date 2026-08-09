/// Modular powers and residues.
///
/// This file develops the applications of Fermat's little theorem and Euler's
/// theorem to modular powers: the power form of Fermat, the last digits of
/// powers of 7 (the period-4 pattern), repeated squaring, and the reduction
/// of exponents modulo p - 1.
///
/// Fermat's little theorem (fermats_little_congr, fermat_euler) and Euler's
/// theorem (euler) are proved in fermat.ac and totient.ac respectively; this
/// file reuses them and proves their power-theoretic corollaries.

from nat import Nat
from nat import exp_add, exp_mul, exp_one, one_exp, pow_pow, small_mod,
    mod_of_decomp, sq_eq_mul, mul_comm, mul_assoc, mul_one_left,
    suc_sub_one, nat_mul_1_4, nat_mul_2_5, nat_mul_4_5, nat_mul_4_6,
    nat_mul_7_7, nat_mul_9_7, nat_mul_9_9, nine_plus_one, nat_add_7_2,
    nat_add_4_3, add_comm, add_suc_right, add_sub, add_cancels_left,
    div_mod_decomp, div_imp_mod, mod_of_zero, mod_lt, lt_add_suc, lt_imp_lt_suc,
    lt_suc, lt_trans
from number_theory.congruence import congr_mod_pow, congr_mod_mul, congr_mod_refl,
    congr_mod_trans, congr_mod_symm, mod_add_eq, divides_of_congr_mod_zero
from number_theory.fermat import fermats_little_congr, fermats_little, fermat_euler,
    rsa_pow_congr
from number_theory.totient import euler, totient_pq, coprime_below_prime
from number_theory.coprime import coprime_comm, coprime_mul
from number_theory.primitive_root_applications import two_coprime_mod_five
from number_theory.zsigmondy import five_is_prime, seven_is_prime, two_ne_five, lt_three_seven
from number_theory.carmichael import two_is_prime
numerals Nat

// ---------------------------------------------------------------------------
// Fermat's little theorem, power form.
// ---------------------------------------------------------------------------

/// Fermat's little theorem, congruence form: for prime `p` and any `a`,
/// `a^p ≡ a (mod p)`. This is the statement proved in fermat.ac, restated here
/// as the power form used by the applications below.
theorem fermat_power_congr(p: Nat, a: Nat) {
    p.is_prime implies a.pow(p).congr_mod(a, p)
} by {
    fermats_little_congr(p, a)
}

/// Fermat's little theorem, remainder form: for prime `p` and any `a`, the
/// remainder of `a^p` modulo `p` equals the remainder of `a` modulo `p`.
theorem fermat_power_mod(p: Nat, a: Nat) {
    p.is_prime implies a.pow(p).mod(p) = a.mod(p)
} by {
    fermats_little(p, a)
}

/// Fermat's little theorem for units: for prime `p` and `a` coprime to `p`,
/// `a^(p - 1) ≡ 1 (mod p)`. This is `fermat_euler` from fermat.ac, restated.
theorem fermat_unit_power_congr(p: Nat, a: Nat) {
    p.is_prime and a.coprime(p) implies a.pow(p - Nat.1).congr_mod(Nat.1, p)
} by {
    fermat_euler(p, a)
}

/// Fermat's little theorem for units, remainder form: for prime `p` and `a`
/// coprime to `p`, the remainder of `a^(p - 1)` modulo `p` is `1`.
theorem fermat_unit_power_mod(p: Nat, a: Nat) {
    p.is_prime and a.coprime(p) implies a.pow(p - Nat.1).mod(p) = Nat.1
} by {
    if p.is_prime and a.coprime(p) {
        fermat_unit_power_congr(p, a)
        a.pow(p - Nat.1).congr_mod(Nat.1, p)
        small_mod(Nat.1, p)
        Nat.1 < p
        Nat.1.mod(p) = Nat.1
        a.pow(p - Nat.1).mod(p) = Nat.1
    }
}

/// RSA exponent congruence: for prime `p`, exponents of the form
/// `k * (p - 1) + 1` behave like the exponent `1` modulo `p`, i.e.
/// `a^(k(p-1)+1) ≡ a (mod p)`. This is `rsa_pow_congr` from fermat.ac,
/// restated as the exponent-reduction form used below.
theorem fermat_exponent_step_congr(p: Nat, a: Nat, k: Nat) {
    p.is_prime implies a.pow(k * (p - Nat.1) + Nat.1).congr_mod(a, p)
} by {
    rsa_pow_congr(p, a, k)
}

// ---------------------------------------------------------------------------
// Euler's theorem.
// ---------------------------------------------------------------------------

/// Euler's theorem: for `n != 0` and `a` coprime to `n`,
/// `a^φ(n) ≡ 1 (mod n)`. This is `euler` from totient.ac, restated.
theorem euler_theorem(n: Nat, a: Nat) {
    n != Nat.0 and a.coprime(n) implies a.pow(n.totient).congr_mod(Nat.1, n)
} by {
    euler(n, a)
}

/// Euler's theorem, remainder form: for `1 < n` and `a` coprime to `n`, the
/// remainder of `a^φ(n)` modulo `n` is `1`.
theorem euler_theorem_mod(n: Nat, a: Nat) {
    Nat.1 < n and a.coprime(n) implies a.pow(n.totient).mod(n) = Nat.1
} by {
    if Nat.1 < n and a.coprime(n) {
        euler_theorem(n, a)
        a.pow(n.totient).congr_mod(Nat.1, n)
        small_mod(Nat.1, n)
        Nat.1.mod(n) = Nat.1
        a.pow(n.totient).mod(n) = Nat.1
    }
}

// ---------------------------------------------------------------------------
// Repeated squaring.
// ---------------------------------------------------------------------------

/// Repeated squaring: `(a²)^k = a^(2k)`. The basis of fast exponentiation:
/// squaring `k` times raises the exponent by a factor of `2^k`.
theorem repeated_squaring(a: Nat, k: Nat) {
    a.pow(Nat.2).pow(k) = a.pow(Nat.2 * k)
} by {
    pow_pow[Nat](a, Nat.2, k)
}

/// Squaring a power: `(a^k)² = a^(2k)`. The other orientation of repeated
/// squaring, obtained by commuting the factors in the exponent.
theorem power_squared(a: Nat, k: Nat) {
    a.pow(k).pow(Nat.2) = a.pow(Nat.2 * k)
} by {
    pow_pow[Nat](a, k, Nat.2)
    a.pow(k * Nat.2) = a.pow(Nat.2 * k)
}

// ---------------------------------------------------------------------------
// The last digits of powers of 7 (period 4 modulo 10).
// ---------------------------------------------------------------------------

/// The digit one is less than ten.
theorem one_lt_ten {
    Nat.1 < Nat.10
} by {
    lt_add_suc(Nat.1, Nat.8)
    Nat.1 < Nat.1 + Nat.9
    add_comm(Nat.9, Nat.1)
    Nat.9 + Nat.1 = Nat.1 + Nat.9
    nine_plus_one
    Nat.9 + Nat.1 = Nat.10
    Nat.1 + Nat.9 = Nat.10
    Nat.1 < Nat.10
}

/// The digit three is less than ten.
theorem three_lt_ten {
    Nat.3 < Nat.10
} by {
    lt_suc(Nat.3)
    Nat.3 < Nat.4
    lt_imp_lt_suc(Nat.3, Nat.4)
    Nat.3 < Nat.5
    lt_imp_lt_suc(Nat.3, Nat.5)
    Nat.3 < Nat.6
    lt_imp_lt_suc(Nat.3, Nat.6)
    Nat.3 < Nat.7
    lt_imp_lt_suc(Nat.3, Nat.7)
    Nat.3 < Nat.8
    lt_imp_lt_suc(Nat.3, Nat.8)
    Nat.3 < Nat.9
    lt_imp_lt_suc(Nat.3, Nat.9)
    Nat.3 < Nat.10
}

/// The digit seven is less than ten.
theorem seven_lt_ten {
    Nat.7 < Nat.10
} by {
    lt_add_suc(Nat.7, Nat.2)
    Nat.7 < Nat.7 + Nat.3
    add_suc_right(Nat.7, Nat.2)
    Nat.7 + Nat.3 = (Nat.7 + Nat.2).suc
    nat_add_7_2
    Nat.7 + Nat.2 = Nat.9
    (Nat.7 + Nat.2).suc = Nat.9.suc
    Nat.9.suc = Nat.10
    Nat.7 + Nat.3 = Nat.10
    Nat.7 < Nat.10
}

/// The digit nine is less than ten.
theorem nine_lt_ten {
    Nat.9 < Nat.10
} by {
    lt_add_suc(Nat.9, Nat.0)
    Nat.9 < Nat.9 + Nat.1
    nine_plus_one
    Nat.9 + Nat.1 = Nat.10
    Nat.9 < Nat.10
}

/// Small case: the last digit of `7^1 = 7` is `7`.
theorem seven_pow_one_last_digit {
    Nat.7.pow(Nat.1).mod(Nat.10) = Nat.7
} by {
    exp_one(Nat.7)
    Nat.7.pow(Nat.1) = Nat.7
    seven_lt_ten
    Nat.7 < Nat.10
    small_mod(Nat.7, Nat.10)
    Nat.7.mod(Nat.10) = Nat.7
    Nat.7.pow(Nat.1).mod(Nat.10) = Nat.7
}

/// Small case: `7² = 49 ≡ 9 (mod 10)`.
theorem seven_pow_two_congr_nine {
    Nat.7.pow(Nat.2).congr_mod(Nat.9, Nat.10)
} by {
    sq_eq_mul(Nat.7)
    Nat.7.pow(Nat.2) = Nat.7 * Nat.7
    nat_mul_7_7
    Nat.7 * Nat.7 = Nat.49
    Nat.7.pow(Nat.2) = Nat.49
    Nat.4 * Nat.10 + Nat.9 = Nat.49
    nine_lt_ten
    Nat.9 < Nat.10
    mod_of_decomp(Nat.4, Nat.9, Nat.10)
    (Nat.4 * Nat.10 + Nat.9).mod(Nat.10) = Nat.9
    Nat.49.mod(Nat.10) = Nat.9
    Nat.7.pow(Nat.2).mod(Nat.10) = Nat.9
    nine_lt_ten
    Nat.9 < Nat.10
    small_mod(Nat.9, Nat.10)
    Nat.9.mod(Nat.10) = Nat.9
    Nat.7.pow(Nat.2).mod(Nat.10) = Nat.9.mod(Nat.10)
}

/// Small case: `7³ = 7² · 7 ≡ 9 · 7 = 63 ≡ 3 (mod 10)`.
theorem seven_pow_three_congr_three {
    Nat.7.pow(Nat.3).congr_mod(Nat.3, Nat.10)
} by {
    exp_add(Nat.7, Nat.2, Nat.1)
    Nat.7.pow(Nat.2 + Nat.1) = Nat.7.pow(Nat.2) * Nat.7.pow(Nat.1)
    exp_one(Nat.7)
    Nat.7.pow(Nat.1) = Nat.7
    Nat.7.pow(Nat.3) = Nat.7.pow(Nat.2) * Nat.7
    seven_pow_two_congr_nine
    Nat.7.pow(Nat.2).congr_mod(Nat.9, Nat.10)
    congr_mod_refl(Nat.7, Nat.10)
    congr_mod_mul(Nat.7.pow(Nat.2), Nat.7, Nat.9, Nat.7, Nat.10)
    (Nat.7.pow(Nat.2) * Nat.7).congr_mod(Nat.9 * Nat.7, Nat.10)
    Nat.7.pow(Nat.3).congr_mod(Nat.9 * Nat.7, Nat.10)
    nat_mul_9_7
    Nat.9 * Nat.7 = Nat.63
    Nat.7.pow(Nat.3).congr_mod(Nat.63, Nat.10)
    Nat.6 * Nat.10 + Nat.3 = Nat.63
    three_lt_ten
    Nat.3 < Nat.10
    mod_of_decomp(Nat.6, Nat.3, Nat.10)
    (Nat.6 * Nat.10 + Nat.3).mod(Nat.10) = Nat.3
    Nat.63.mod(Nat.10) = Nat.3
    Nat.7.pow(Nat.3).mod(Nat.10) = Nat.63.mod(Nat.10)
    Nat.7.pow(Nat.3).mod(Nat.10) = Nat.3
    three_lt_ten
    Nat.3 < Nat.10
    small_mod(Nat.3, Nat.10)
    Nat.3.mod(Nat.10) = Nat.3
    Nat.7.pow(Nat.3).mod(Nat.10) = Nat.3.mod(Nat.10)
    Nat.7.pow(Nat.3).congr_mod(Nat.3, Nat.10)
}

/// Small case: `7⁴ = (7²)² ≡ 9² = 81 ≡ 1 (mod 10)`. Since `7⁴ ≡ 1`, the
/// powers of 7 repeat every four steps.
theorem seven_pow_four_congr_one {
    Nat.7.pow(Nat.4).congr_mod(Nat.1, Nat.10)
} by {
    exp_mul(Nat.7, Nat.2, Nat.2)
    Nat.7.pow(Nat.2 * Nat.2) = Nat.7.pow(Nat.2).pow(Nat.2)
    Nat.2 * Nat.2 = Nat.4
    Nat.7.pow(Nat.4) = Nat.7.pow(Nat.2).pow(Nat.2)
    seven_pow_two_congr_nine
    Nat.7.pow(Nat.2).congr_mod(Nat.9, Nat.10)
    congr_mod_pow(Nat.7.pow(Nat.2), Nat.9, Nat.10, Nat.2)
    Nat.7.pow(Nat.2).pow(Nat.2).congr_mod(Nat.9.pow(Nat.2), Nat.10)
    Nat.7.pow(Nat.4).congr_mod(Nat.9.pow(Nat.2), Nat.10)
    sq_eq_mul(Nat.9)
    Nat.9.pow(Nat.2) = Nat.9 * Nat.9
    nat_mul_9_9
    Nat.9 * Nat.9 = Nat.81
    Nat.9.pow(Nat.2) = Nat.81
    Nat.7.pow(Nat.4).congr_mod(Nat.81, Nat.10)
    Nat.8 * Nat.10 + Nat.1 = Nat.81
    one_lt_ten
    Nat.1 < Nat.10
    mod_of_decomp(Nat.8, Nat.1, Nat.10)
    (Nat.8 * Nat.10 + Nat.1).mod(Nat.10) = Nat.1
    Nat.81.mod(Nat.10) = Nat.1
    Nat.7.pow(Nat.4).mod(Nat.10) = Nat.81.mod(Nat.10)
    Nat.7.pow(Nat.4).mod(Nat.10) = Nat.1
    one_lt_ten
    Nat.1 < Nat.10
    small_mod(Nat.1, Nat.10)
    Nat.1.mod(Nat.10) = Nat.1
    Nat.7.pow(Nat.4).mod(Nat.10) = Nat.1.mod(Nat.10)
    Nat.7.pow(Nat.4).congr_mod(Nat.1, Nat.10)
}

/// The period of 7 modulo 10: for every `k`, `7^(4k) ≡ 1 (mod 10)`.
theorem seven_pow_period(k: Nat) {
    Nat.7.pow(Nat.4 * k).congr_mod(Nat.1, Nat.10)
} by {
    seven_pow_four_congr_one
    Nat.7.pow(Nat.4).congr_mod(Nat.1, Nat.10)
    congr_mod_pow(Nat.7.pow(Nat.4), Nat.1, Nat.10, k)
    Nat.7.pow(Nat.4).pow(k).congr_mod(Nat.1.pow(k), Nat.10)
    exp_mul(Nat.7, Nat.4, k)
    Nat.7.pow(Nat.4 * k) = Nat.7.pow(Nat.4).pow(k)
    one_exp(k)
    Nat.1.pow(k) = Nat.1
    Nat.7.pow(Nat.4).pow(k).congr_mod(Nat.1, Nat.10)
    Nat.7.pow(Nat.4 * k).congr_mod(Nat.1, Nat.10)
}

/// The residue of a power of 7 modulo 10 is determined by the exponent modulo
/// 4: `7^(4k + r) ≡ 7^r (mod 10)` for all `k` and `r`.
theorem seven_pow_period_residue(k: Nat, r: Nat) {
    Nat.7.pow(Nat.4 * k + r).congr_mod(Nat.7.pow(r), Nat.10)
} by {
    exp_add(Nat.7, Nat.4 * k, r)
    Nat.7.pow(Nat.4 * k + r) = Nat.7.pow(Nat.4 * k) * Nat.7.pow(r)
    seven_pow_period(k)
    Nat.7.pow(Nat.4 * k).congr_mod(Nat.1, Nat.10)
    congr_mod_refl(Nat.7.pow(r), Nat.10)
    congr_mod_mul(Nat.7.pow(Nat.4 * k), Nat.7.pow(r), Nat.1, Nat.7.pow(r), Nat.10)
    (Nat.7.pow(Nat.4 * k) * Nat.7.pow(r)).congr_mod(Nat.1 * Nat.7.pow(r), Nat.10)
    Nat.1 * Nat.7.pow(r) = Nat.7.pow(r)
    Nat.7.pow(Nat.4 * k + r).congr_mod(Nat.7.pow(r), Nat.10)
}

/// General pattern: exponents divisible by 4 leave last digit 1.
theorem seven_pow_four_k_last_digit(k: Nat) {
    Nat.7.pow(Nat.4 * k).mod(Nat.10) = Nat.1
} by {
    seven_pow_period(k)
    Nat.7.pow(Nat.4 * k).congr_mod(Nat.1, Nat.10)
    one_lt_ten
    Nat.1 < Nat.10
    small_mod(Nat.1, Nat.10)
    Nat.1.mod(Nat.10) = Nat.1
    Nat.7.pow(Nat.4 * k).mod(Nat.10) = Nat.1
}

/// General pattern: exponents congruent to 1 modulo 4 leave last digit 7.
theorem seven_pow_four_k_plus_one_last_digit(k: Nat) {
    Nat.7.pow(Nat.4 * k + Nat.1).mod(Nat.10) = Nat.7
} by {
    seven_pow_period_residue(k, Nat.1)
    Nat.7.pow(Nat.4 * k + Nat.1).congr_mod(Nat.7.pow(Nat.1), Nat.10)
    exp_one(Nat.7)
    Nat.7.pow(Nat.1) = Nat.7
    Nat.7.pow(Nat.4 * k + Nat.1).congr_mod(Nat.7, Nat.10)
    seven_lt_ten
    Nat.7 < Nat.10
    small_mod(Nat.7, Nat.10)
    Nat.7.mod(Nat.10) = Nat.7
    Nat.7.pow(Nat.4 * k + Nat.1).mod(Nat.10) = Nat.7
}

/// General pattern: exponents congruent to 2 modulo 4 leave last digit 9.
theorem seven_pow_four_k_plus_two_last_digit(k: Nat) {
    Nat.7.pow(Nat.4 * k + Nat.2).mod(Nat.10) = Nat.9
} by {
    seven_pow_period_residue(k, Nat.2)
    Nat.7.pow(Nat.4 * k + Nat.2).congr_mod(Nat.7.pow(Nat.2), Nat.10)
    seven_pow_two_congr_nine
    Nat.7.pow(Nat.2).congr_mod(Nat.9, Nat.10)
    congr_mod_trans(Nat.7.pow(Nat.4 * k + Nat.2), Nat.7.pow(Nat.2), Nat.9, Nat.10)
    Nat.7.pow(Nat.4 * k + Nat.2).congr_mod(Nat.9, Nat.10)
    nine_lt_ten
    Nat.9 < Nat.10
    small_mod(Nat.9, Nat.10)
    Nat.9.mod(Nat.10) = Nat.9
    Nat.7.pow(Nat.4 * k + Nat.2).mod(Nat.10) = Nat.9
}

/// General pattern: exponents congruent to 3 modulo 4 leave last digit 3.
theorem seven_pow_four_k_plus_three_last_digit(k: Nat) {
    Nat.7.pow(Nat.4 * k + Nat.3).mod(Nat.10) = Nat.3
} by {
    seven_pow_period_residue(k, Nat.3)
    Nat.7.pow(Nat.4 * k + Nat.3).congr_mod(Nat.7.pow(Nat.3), Nat.10)
    seven_pow_three_congr_three
    Nat.7.pow(Nat.3).congr_mod(Nat.3, Nat.10)
    congr_mod_trans(Nat.7.pow(Nat.4 * k + Nat.3), Nat.7.pow(Nat.3), Nat.3, Nat.10)
    Nat.7.pow(Nat.4 * k + Nat.3).congr_mod(Nat.3, Nat.10)
    three_lt_ten
    Nat.3 < Nat.10
    small_mod(Nat.3, Nat.10)
    Nat.3.mod(Nat.10) = Nat.3
    Nat.7.pow(Nat.4 * k + Nat.3).mod(Nat.10) = Nat.3
}

/// `506 = 50 · 10 + 6`: the base-ten digits of 506.
theorem five_hundred_six_decomp {
    Nat.50 * Nat.10 + Nat.6 = Nat.506
}

/// `200 = 20 · 10`.
theorem two_hundred_ten_decomp {
    Nat.20 * Nat.10 = Nat.200
}

/// `4 · 50 = 200`.
theorem mul_four_fifty {
    Nat.4 * Nat.50 = Nat.200
} by {
    mul_assoc(Nat.4, Nat.5, Nat.10)
    (Nat.4 * Nat.5) * Nat.10 = Nat.4 * (Nat.5 * Nat.10)
    nat_mul_4_5
    Nat.4 * Nat.5 = Nat.20
    Nat.20 * Nat.10 = Nat.4 * (Nat.5 * Nat.10)
    Nat.5 * Nat.10 = Nat.50
    Nat.20 * Nat.10 = Nat.4 * Nat.50
    two_hundred_ten_decomp
    Nat.20 * Nat.10 = Nat.200
    Nat.4 * Nat.50 = Nat.200
}

/// `2000 = 200 · 10`.
theorem two_thousand_ten_decomp {
    Nat.200 * Nat.10 = Nat.2000
}

/// `2024 = 2000 + 24`.
theorem two_thousand_twenty_four_add {
    Nat.2000 + Nat.24 = Nat.2024
}

/// `2024 = 200 · 10 + 24`.
theorem two_thousand_twenty_four_ten_decomp {
    Nat.200 * Nat.10 + Nat.24 = Nat.2024
} by {
    two_thousand_ten_decomp
    Nat.200 * Nat.10 = Nat.2000
    two_thousand_twenty_four_add
    Nat.2000 + Nat.24 = Nat.2024
    Nat.200 * Nat.10 + Nat.24 = Nat.2024
}

/// `4 · 506 = 2024`, computed through the digit decomposition of 506:
/// `4 · (50 · 10 + 6) = 4 · 50 · 10 + 4 · 6 = 200 · 10 + 24 = 2024`.
theorem mul_four_five_hundred_six {
    Nat.4 * Nat.506 = Nat.2024
} by {
    five_hundred_six_decomp
    Nat.50 * Nat.10 + Nat.6 = Nat.506
    Nat.4 * Nat.506 = Nat.4 * (Nat.50 * Nat.10 + Nat.6)
    Nat.4 * (Nat.50 * Nat.10 + Nat.6) = Nat.4 * (Nat.50 * Nat.10) + Nat.4 * Nat.6
    mul_assoc(Nat.4, Nat.50, Nat.10)
    Nat.4 * (Nat.50 * Nat.10) = (Nat.4 * Nat.50) * Nat.10
    mul_four_fifty
    Nat.4 * Nat.50 = Nat.200
    Nat.4 * (Nat.50 * Nat.10) = Nat.200 * Nat.10
    nat_mul_4_6
    Nat.4 * Nat.6 = Nat.24
    Nat.4 * Nat.506 = Nat.200 * Nat.10 + Nat.24
    two_thousand_twenty_four_ten_decomp
    Nat.200 * Nat.10 + Nat.24 = Nat.2024
    Nat.4 * Nat.506 = Nat.2024
}

/// `2025 = 4 · 506 + 1`: the exponent of `7^2025` is one more than a multiple
/// of four.
theorem two_thousand_twenty_five_decomp {
    Nat.4 * Nat.506 + Nat.1 = Nat.2025
} by {
    mul_four_five_hundred_six
    Nat.4 * Nat.506 = Nat.2024
    Nat.2024 + Nat.1 = Nat.2025
    Nat.4 * Nat.506 + Nat.1 = Nat.2025
}

/// `7^2025 ≡ 7 (mod 10)`: since `2025 = 4 · 506 + 1`, the period-4 pattern
/// gives the same residue as `7^1`.
theorem seven_pow_2025_congr_seven {
    Nat.7.pow(Nat.2025).congr_mod(Nat.7, Nat.10)
} by {
    two_thousand_twenty_five_decomp
    Nat.4 * Nat.506 + Nat.1 = Nat.2025
    seven_pow_period_residue(Nat.506, Nat.1)
    Nat.7.pow(Nat.4 * Nat.506 + Nat.1).congr_mod(Nat.7.pow(Nat.1), Nat.10)
    Nat.7.pow(Nat.2025).congr_mod(Nat.7.pow(Nat.1), Nat.10)
    exp_one(Nat.7)
    Nat.7.pow(Nat.1) = Nat.7
    Nat.7.pow(Nat.2025).congr_mod(Nat.7, Nat.10)
}

/// The last digit of `7^2025` is `7`.
theorem seven_pow_2025_last_digit {
    Nat.7.pow(Nat.2025).mod(Nat.10) = Nat.7
} by {
    seven_pow_2025_congr_seven
    Nat.7.pow(Nat.2025).congr_mod(Nat.7, Nat.10)
    seven_lt_ten
    Nat.7 < Nat.10
    small_mod(Nat.7, Nat.10)
    Nat.7.mod(Nat.10) = Nat.7
    Nat.7.pow(Nat.2025).mod(Nat.10) = Nat.7
}

// ---------------------------------------------------------------------------
// The period 4 via Euler's theorem: 7^φ(10) = 7^4 ≡ 1 (mod 10).
// ---------------------------------------------------------------------------

/// Seven is coprime to ten.
theorem seven_coprime_ten {
    Nat.7.coprime(Nat.10)
} by {
    seven_is_prime
    Nat.7.is_prime
    Nat.1 <= Nat.2
    lt_suc(Nat.2)
    Nat.2 < Nat.3
    lt_three_seven
    Nat.3 < Nat.7
    lt_trans(Nat.2, Nat.3, Nat.7)
    Nat.2 < Nat.7
    coprime_below_prime(Nat.7, Nat.2)
    Nat.2.coprime(Nat.7)
    coprime_comm(Nat.2, Nat.7)
    Nat.7.coprime(Nat.2)
    Nat.1 + Nat.4 = Nat.5
    exists(c: Nat) { Nat.1 + c = Nat.5 }
    Nat.1 <= Nat.5
    lt_suc(Nat.5)
    Nat.5 < Nat.6
    lt_suc(Nat.6)
    Nat.6 < Nat.7
    lt_trans(Nat.5, Nat.6, Nat.7)
    Nat.5 < Nat.7
    coprime_below_prime(Nat.7, Nat.5)
    Nat.5.coprime(Nat.7)
    coprime_comm(Nat.5, Nat.7)
    Nat.7.coprime(Nat.5)
    coprime_mul(Nat.7, Nat.2, Nat.5)
    Nat.7.coprime(Nat.2 * Nat.5)
    nat_mul_2_5
    Nat.2 * Nat.5 = Nat.10
    Nat.7.coprime(Nat.10)
}

/// The totient of ten is four: `φ(10) = (2 - 1)(5 - 1) = 4`.
theorem ten_totient_four {
    Nat.10.totient = Nat.4
} by {
    two_is_prime
    Nat.2.is_prime
    five_is_prime
    Nat.5.is_prime
    two_ne_five
    Nat.2 != Nat.5
    totient_pq(Nat.2, Nat.5)
    (Nat.2 * Nat.5).totient = (Nat.2 - Nat.1) * (Nat.5 - Nat.1)
    nat_mul_2_5
    Nat.2 * Nat.5 = Nat.10
    Nat.10.totient = (Nat.2 - Nat.1) * (Nat.5 - Nat.1)
    suc_sub_one(Nat.1)
    Nat.2 - Nat.1 = Nat.1
    suc_sub_one(Nat.4)
    Nat.5 - Nat.1 = Nat.4
    (Nat.2 - Nat.1) * (Nat.5 - Nat.1) = Nat.1 * Nat.4
    nat_mul_1_4
    Nat.1 * Nat.4 = Nat.4
    Nat.10.totient = Nat.4
}

/// Euler's theorem at the modulus 10: since `7` is coprime to `10` and
/// `φ(10) = 4`, `7^4 ≡ 1 (mod 10)`. This is the same fact as
/// `seven_pow_four_congr_one`, obtained through Euler's theorem.
theorem seven_pow_four_congr_one_via_euler {
    Nat.7.pow(Nat.4).congr_mod(Nat.1, Nat.10)
} by {
    seven_coprime_ten
    Nat.7.coprime(Nat.10)
    ten_totient_four
    Nat.10.totient = Nat.4
    euler(Nat.10, Nat.7)
    Nat.7.pow(Nat.10.totient).congr_mod(Nat.1, Nat.10)
    Nat.7.pow(Nat.4).congr_mod(Nat.1, Nat.10)
}

// ---------------------------------------------------------------------------
// Exponent reduction via Fermat's little theorem.
// ---------------------------------------------------------------------------

/// Exponent reduction (Fermat form): for prime `p` and `a` coprime to `p`, if
/// `m = n + k(p - 1)` then `a^m ≡ a^n (mod p)`. Raising to the `(p - 1)`-th
/// power is trivial modulo `p` by Fermat, so exponents may be reduced modulo
/// `p - 1`.
theorem exponent_reduction_fermat(p: Nat, a: Nat, m: Nat, n: Nat, k: Nat) {
    p.is_prime and a.coprime(p) and m = n + k * (p - Nat.1)
        implies a.pow(m).congr_mod(a.pow(n), p)
} by {
    if p.is_prime and a.coprime(p) and m = n + k * (p - Nat.1) {
        exp_add(a, n, k * (p - Nat.1))
        a.pow(n + k * (p - Nat.1)) = a.pow(n) * a.pow(k * (p - Nat.1))
        a.pow(m) = a.pow(n) * a.pow(k * (p - Nat.1))
        mul_comm(k, p - Nat.1)
        k * (p - Nat.1) = (p - Nat.1) * k
        exp_mul(a, p - Nat.1, k)
        a.pow((p - Nat.1) * k) = a.pow(p - Nat.1).pow(k)
        a.pow(k * (p - Nat.1)) = a.pow(p - Nat.1).pow(k)
        a.pow(m) = a.pow(n) * a.pow(p - Nat.1).pow(k)
        fermat_euler(p, a)
        a.pow(p - Nat.1).congr_mod(Nat.1, p)
        congr_mod_pow(a.pow(p - Nat.1), Nat.1, p, k)
        a.pow(p - Nat.1).pow(k).congr_mod(Nat.1.pow(k), p)
        one_exp(k)
        Nat.1.pow(k) = Nat.1
        a.pow(p - Nat.1).pow(k).congr_mod(Nat.1, p)
        congr_mod_refl(a.pow(n), p)
        congr_mod_mul(a.pow(n), a.pow(p - Nat.1).pow(k), a.pow(n), Nat.1, p)
        (a.pow(n) * a.pow(p - Nat.1).pow(k)).congr_mod(a.pow(n) * Nat.1, p)
        a.pow(n) * Nat.1 = a.pow(n)
        (a.pow(n) * a.pow(p - Nat.1).pow(k)).congr_mod(a.pow(n), p)
        a.pow(m).congr_mod(a.pow(n), p)
    }
}

/// Two is coprime to five.
theorem two_coprime_five {
    Nat.2.coprime(Nat.5)
} by {
    two_coprime_mod_five
}

/// Concrete exponent reduction: `7 = 3 + 1 · 4` and `2` is coprime to `5`, so
/// `2^7 ≡ 2^3 (mod 5)`.
theorem two_pow_seven_congr_two_pow_three_mod_five {
    Nat.2.pow(Nat.7).congr_mod(Nat.2.pow(Nat.3), Nat.5)
} by {
    five_is_prime
    Nat.5.is_prime
    two_coprime_five
    Nat.2.coprime(Nat.5)
    suc_sub_one(Nat.4)
    Nat.5 - Nat.1 = Nat.4
    mul_one_left(Nat.4)
    Nat.1 * Nat.4 = Nat.4
    Nat.3 + Nat.4 = Nat.7
    Nat.3 + Nat.1 * (Nat.5 - Nat.1) = Nat.7
    exponent_reduction_fermat(Nat.5, Nat.2, Nat.7, Nat.3, Nat.1)
    Nat.2.pow(Nat.7).congr_mod(Nat.2.pow(Nat.3), Nat.5)
}

// ---------------------------------------------------------------------------
// Exponent reduction in congruence form.
// ---------------------------------------------------------------------------

/// For a nonzero modulus `q`, if `r < q` and `(r + s) mod q = r`, then
/// `s mod q = 0`. Cancellation of a bounded remainder in modular addition.
theorem add_mod_cancel(q: Nat, r: Nat, s: Nat) {
    r < q and (r + s).mod(q) = r implies s.mod(q) = Nat.0
} by {
    if r < q and (r + s).mod(q) = r {
        div_mod_decomp(r + s, q)
        (r + s).div(q) * q + (r + s).mod(q) = r + s
        (r + s).div(q) * q + r = r + s
        add_comm((r + s).div(q) * q, r)
        r + (r + s).div(q) * q = r + s
        add_cancels_left(r, (r + s).div(q) * q, s)
        (r + s).div(q) * q = s
        mul_comm((r + s).div(q), q)
        (r + s).div(q) * q = q * (r + s).div(q)
        q * (r + s).div(q) = s
        exists(c: Nat) { q * c = s }
        q.divides(s)
        div_imp_mod(s, q)
        s.mod(q) = Nat.0
    }
}

/// Bridging lemma: if `n <= m` and `m ≡ n (mod q)` with `q != 0`, then
/// `m = n + kq` for some `k`. This converts the congruence `m ≡ n (mod q)`
/// into the difference form used by exponent reduction.
theorem congr_mod_lte_imp_exists_k(m: Nat, n: Nat, q: Nat) {
    n <= m and m.congr_mod(n, q) and q != Nat.0 implies exists(k: Nat) { m = n + k * q }
} by {
    if n <= m and m.congr_mod(n, q) and q != Nat.0 {
        add_sub(m, n)
        m - n + n = m
        add_comm(m - n, n)
        n + (m - n) = m
        mod_add_eq(n, m - n, q)
        (n + (m - n)).mod(q) = (n.mod(q) + (m - n).mod(q)).mod(q)
        m.mod(q) = (n.mod(q) + (m - n).mod(q)).mod(q)
        m.mod(q) = n.mod(q)
        n.mod(q) = (n.mod(q) + (m - n).mod(q)).mod(q)
        mod_lt(n, q)
        n.mod(q) < q
        add_mod_cancel(q, n.mod(q), m - n)
        (m - n).mod(q) = Nat.0
        mod_of_zero(q)
        Nat.0.mod(q) = Nat.0
        (m - n).mod(q) = Nat.0.mod(q)
        (m - n).congr_mod(Nat.0, q)
        divides_of_congr_mod_zero(q, m - n)
        q.divides(m - n)
        q.divides(m - n) = exists(c: Nat) { q * c = m - n }
        exists(c: Nat) { q * c = m - n }
        let kk: Nat satisfy { q * kk = m - n }
        q * kk = m - n
        mul_comm(q, kk)
        kk * q = m - n
        n + kk * q = m
        exists(j: Nat) { m = n + j * q }
    }
}

/// For a prime `p`, `p - 1` is nonzero.
theorem prime_sub_one_ne_zero(p: Nat) {
    p.is_prime implies p - Nat.1 != Nat.0
} by {
    if p.is_prime {
        Nat.1 < p
        if p - Nat.1 = Nat.0 {
            add_sub(p, Nat.1)
            Nat.1 <= p
            p - Nat.1 + Nat.1 = p
            p = Nat.1
            false
        }
    }
}

/// Exponent reduction, congruence form: for prime `p` and `a` coprime to `p`,
/// if `m ≡ n (mod p - 1)` then `a^m ≡ a^n (mod p)`. This is the statement of
/// the Fermat exponent reduction with the difference hypothesis `m = n + k(p-1)`
/// replaced by the congruence `m ≡ n (mod p - 1)`.
theorem exponent_reduction_congr(p: Nat, a: Nat, m: Nat, n: Nat) {
    p.is_prime and a.coprime(p) and m.congr_mod(n, p - Nat.1)
        implies a.pow(m).congr_mod(a.pow(n), p)
} by {
    if p.is_prime and a.coprime(p) and m.congr_mod(n, p - Nat.1) {
        prime_sub_one_ne_zero(p)
        p - Nat.1 != Nat.0
        if n <= m {
            congr_mod_lte_imp_exists_k(m, n, p - Nat.1)
            exists(k: Nat) { m = n + k * (p - Nat.1) }
            let kk: Nat satisfy { m = n + kk * (p - Nat.1) }
            m = n + kk * (p - Nat.1)
            exponent_reduction_fermat(p, a, m, n, kk)
            a.pow(m).congr_mod(a.pow(n), p)
        } else {
            m < n
            m <= n
            congr_mod_symm(m, n, p - Nat.1)
            n.congr_mod(m, p - Nat.1)
            congr_mod_lte_imp_exists_k(n, m, p - Nat.1)
            exists(k: Nat) { n = m + k * (p - Nat.1) }
            let kk: Nat satisfy { n = m + kk * (p - Nat.1) }
            n = m + kk * (p - Nat.1)
            exponent_reduction_fermat(p, a, n, m, kk)
            a.pow(n).congr_mod(a.pow(m), p)
            congr_mod_symm(a.pow(n), a.pow(m), p)
            a.pow(m).congr_mod(a.pow(n), p)
        }
    }
}

/// `7 ≡ 3 (mod 4)`: since `7 = 1 · 4 + 3`.
theorem seven_congr_three_mod_four {
    Nat.7.congr_mod(Nat.3, Nat.4)
} by {
    nat_mul_1_4
    Nat.1 * Nat.4 = Nat.4
    nat_add_4_3
    Nat.4 + Nat.3 = Nat.7
    Nat.1 * Nat.4 + Nat.3 = Nat.7
    lt_suc(Nat.3)
    Nat.3 < Nat.4
    mod_of_decomp(Nat.1, Nat.3, Nat.4)
    (Nat.1 * Nat.4 + Nat.3).mod(Nat.4) = Nat.3
    Nat.7.mod(Nat.4) = Nat.3
    small_mod(Nat.3, Nat.4)
    Nat.3.mod(Nat.4) = Nat.3
    Nat.7.mod(Nat.4) = Nat.3.mod(Nat.4)
}

/// Concrete exponent reduction through the congruence form: since `7 ≡ 3
/// (mod 4)` and `2` is coprime to `5`, `2^7 ≡ 2^3 (mod 5)`.
theorem two_pow_seven_congr_two_pow_three_mod_five_congr {
    Nat.2.pow(Nat.7).congr_mod(Nat.2.pow(Nat.3), Nat.5)
} by {
    five_is_prime
    Nat.5.is_prime
    two_coprime_five
    Nat.2.coprime(Nat.5)
    seven_congr_three_mod_four
    Nat.7.congr_mod(Nat.3, Nat.4)
    exponent_reduction_congr(Nat.5, Nat.2, Nat.7, Nat.3)
    Nat.2.pow(Nat.7).congr_mod(Nat.2.pow(Nat.3), Nat.5)
}
