from nat import Nat
from list import List, map, sum, partial, partial_split_last
from nat import divides_self, exp_zero, exp_ne_zero, exp_one, exp_add,
    lt_not_ref, trichotomy, lt_suc, lte_and_lt, lte_add_left,
    add_sub, add_imp_sub, suc_sub_one, sub_pos, pos_of_ne_zero,
    zero_or_suc, lt_suc_right, mul_to_zero, lte_imp_not_lt, lt_imp_lte_suc,
    not_lt_zero, div_mul, only_zero_lte_zero, lt_trans, lte_trans,
    add_assoc, add_comm
from list import unique_same_contains_map_sum_eq,
    map_contains, map_contains_of_contains, range_contains_of_lt,
    range_contains_iff_lt, range_is_unique, injective_map_is_unique,
    sum_map_remove_one, remove_one_contains_other
from number_theory.factorisation import count_prime_factor, count_prime_factor_pow,
    count_prime_factor_pow_other, count_prime_factor_ext,
    divides_imp_count_prime_factor_le, no_proper_divisor_imp_prime
from number_theory.divisor_sum import divisor_list, nat_sigma, divisor_list_is_unique,
    divisor_list_contains_implies, divisor_list_contains_of, nat_sigma_prime,
    nat_sigma_zero, nat_sigma_one, sum_map_nat_identity_arithmetic_fn_eq_sum,
    one_divides_nat
from number_theory.arithmetic_functions import nat_identity_arithmetic_fn,
    nat_identity_arithmetic_fn_apply
from number_theory.dirichlet import nat_sigma_mul_coprime_positive
from data.basic.functions import is_injective_fn
numerals Nat

/// Subtracting one from a positive number and adding it back is the identity:
/// `(p - 1) + 1 = p`.
theorem sub_one_add_one(p: Nat) {
    Nat.1 <= p implies (p - Nat.1) + Nat.1 = p
} by {
    if Nat.1 <= p {
        if p = Nat.0 {
            Nat.1 <= Nat.0
            only_zero_lte_zero(Nat.1)
            Nat.1 = Nat.0
            false
        }
        p != Nat.0
        zero_or_suc(p)
        let q: Nat satisfy { q.suc = p }
        suc_sub_one(q)
        q.suc - Nat.1 = q
        (p - Nat.1) + Nat.1 = q + Nat.1
        q + Nat.1 = q.suc
        (p - Nat.1) + Nat.1 = p
    }
}

/// A prime power `p^(k+1)` splits as `p^i * p^(k-i)` whenever `i <= k`.
theorem pow_sub_mul(p: Nat, i: Nat, k: Nat) {
    i <= k implies p.pow(i) * p.pow(k - i) = p.pow(k)
} by {
    if i <= k {
        exp_add(p, i, k - i)
        p.pow(i + (k - i)) = p.pow(i) * p.pow(k - i)
        add_sub(i, k)
        k - i + i = k
        i + (k - i) = k
        p.pow(i + (k - i)) = p.pow(k)
        p.pow(i) * p.pow(k - i) = p.pow(k)
    }
}

/// The power map `x -> p^x` is injective for a prime `p`: equal powers have
/// equal exponents, since the prime count of `p^i` is `i`.
theorem nat_pow_injective(p: Nat) {
    p.is_prime implies is_injective_fn[Nat, Nat](p.pow)
} by {
    if p.is_prime {
        is_injective_fn[Nat, Nat](p.pow) = forall(x: Nat, y: Nat) {
            p.pow(x) = p.pow(y) implies x = y
        }
        forall(i: Nat, j: Nat) {
            if p.pow(i) = p.pow(j) {
                Nat.1 < p
                p != Nat.0
                exp_ne_zero(p, i)
                p.pow(i) != Nat.0
                exp_ne_zero(p, j)
                p.pow(j) != Nat.0
                count_prime_factor_pow(p, i)
                count_prime_factor(p, p.pow(i)) = i
                count_prime_factor_pow(p, j)
                count_prime_factor(p, p.pow(j)) = j
                count_prime_factor(p, p.pow(i)) =
                    count_prime_factor(p, p.pow(j))
                i = j
            }
        }
        is_injective_fn[Nat, Nat](p.pow)
    }
}

/// The list `[p^0, p^1, ..., p^k]` of powers of a prime has no repetitions.
theorem pow_range_list_unique(p: Nat, k: Nat) {
    p.is_prime implies map((k + Nat.1).range, p.pow).is_unique
} by {
    if p.is_prime {
        range_is_unique(k + Nat.1)
        (k + Nat.1).range.is_unique
        nat_pow_injective(p)
        is_injective_fn[Nat, Nat](p.pow)
        injective_map_is_unique((k + Nat.1).range, p.pow)
        map((k + Nat.1).range, p.pow).is_unique
    }
}

/// Every positive divisor of a prime power is itself a power of the prime.
theorem prime_pow_divisor_eq_pow(p: Nat, k: Nat, d: Nat) {
    p.is_prime and d.divides(p.pow(k)) and d != Nat.0
        implies exists(i: Nat) { i <= k and d = p.pow(i) }
} by {
    if p.is_prime and d.divides(p.pow(k)) and d != Nat.0 {
        Nat.1 < p
        p != Nat.0
        exp_ne_zero(p, k)
        p.pow(k) != Nat.0
        divides_imp_count_prime_factor_le(p, d, p.pow(k))
        count_prime_factor(p, d) <= count_prime_factor(p, p.pow(k))
        count_prime_factor_pow(p, k)
        count_prime_factor(p, p.pow(k)) = k
        count_prime_factor(p, d) <= k
        let m: Nat = count_prime_factor(p, d)
        m <= k
        exp_ne_zero(p, m)
        p.pow(m) != Nat.0
        forall(q: Nat) {
            if q.is_prime {
                if q = p {
                    count_prime_factor(q, d) = count_prime_factor(p, d)
                    count_prime_factor_pow(p, m)
                    count_prime_factor(p, p.pow(m)) = m
                    count_prime_factor(q, d) = count_prime_factor(q, p.pow(m))
                } else {
                    q != p
                    divides_imp_count_prime_factor_le(q, d, p.pow(k))
                    count_prime_factor(q, d) <= count_prime_factor(q, p.pow(k))
                    count_prime_factor_pow_other(p, q, k)
                    count_prime_factor(q, p.pow(k)) = Nat.0
                    count_prime_factor(q, d) = Nat.0
                    count_prime_factor_pow_other(p, q, m)
                    count_prime_factor(q, p.pow(m)) = Nat.0
                    count_prime_factor(q, d) = count_prime_factor(q, p.pow(m))
                }
            }
        }
        count_prime_factor_ext(d, p.pow(m))
        d = p.pow(m)
        m <= k
        exists(i: Nat) { i <= k and d = p.pow(i) }
    }
}

/// Every divisor of `p^k` lies among the powers `p^0, ..., p^k`.
theorem divisor_list_prime_pow_contains(p: Nat, k: Nat, x: Nat) {
    p.is_prime and divisor_list(p.pow(k)).contains(x)
        implies map((k + Nat.1).range, p.pow).contains(x)
} by {
    if p.is_prime and divisor_list(p.pow(k)).contains(x) {
        divisor_list_contains_implies(p.pow(k), x)
        Nat.0 < x
        x.divides(p.pow(k))
        x != Nat.0
        prime_pow_divisor_eq_pow(p, k, x)
        let i: Nat satisfy { i <= k and x = p.pow(i) }
        i <= k
        lt_suc(k)
        k < k.suc
        lte_and_lt(i, k, k.suc)
        i < k.suc
        k.suc = k + Nat.1
        i < k + Nat.1
        range_contains_of_lt(k + Nat.1, i)
        (k + Nat.1).range.contains(i)
        map_contains_of_contains((k + Nat.1).range, p.pow, i)
        map((k + Nat.1).range, p.pow).contains(p.pow(i))
        x = p.pow(i)
        map((k + Nat.1).range, p.pow).contains(x)
    }
}

/// Conversely, every power `p^i` with `i <= k` divides `p^k`.
theorem divisor_list_prime_pow_contains_of(p: Nat, k: Nat, x: Nat) {
    p.is_prime and map((k + Nat.1).range, p.pow).contains(x)
        implies divisor_list(p.pow(k)).contains(x)
} by {
    if p.is_prime and map((k + Nat.1).range, p.pow).contains(x) {
        Nat.1 < p
        p != Nat.0
        map_contains((k + Nat.1).range, p.pow, x)
        let i: Nat satisfy {
            (k + Nat.1).range.contains(i) and p.pow(i) = x
        }
        range_contains_iff_lt(k + Nat.1, i)
        (k + Nat.1).range.contains(i) = (i < k + Nat.1)
        i < k + Nat.1
        k + Nat.1 = k.suc
        i < k.suc
        lt_suc_right(i, k)
        i = k or i < k
        i <= k
        pow_sub_mul(p, i, k)
        p.pow(i) * p.pow(k - i) = p.pow(k)
        p.pow(i).divides(p.pow(k))
        x.divides(p.pow(k))
        exp_ne_zero(p, i)
        p.pow(i) != Nat.0
        pos_of_ne_zero(p.pow(i))
        Nat.0 < p.pow(i)
        x = p.pow(i)
        Nat.0 < x
        exp_ne_zero(p, k)
        p.pow(k) != Nat.0
        pos_of_ne_zero(p.pow(k))
        Nat.0 < p.pow(k)
        divisor_list_contains_of(p.pow(k), x)
        divisor_list(p.pow(k)).contains(x)
    }
}

/// The divisor list of `p^k` and the power list `[p^0, ..., p^k]` have the
/// same members.
theorem divisor_list_prime_pow_contains_iff(p: Nat, k: Nat, x: Nat) {
    p.is_prime implies (
        divisor_list(p.pow(k)).contains(x) =
            map((k + Nat.1).range, p.pow).contains(x)
    )
} by {
    if p.is_prime {
        if divisor_list(p.pow(k)).contains(x) {
            divisor_list_prime_pow_contains(p, k, x)
            map((k + Nat.1).range, p.pow).contains(x)
        }
        if map((k + Nat.1).range, p.pow).contains(x) {
            divisor_list_prime_pow_contains_of(p, k, x)
            divisor_list(p.pow(k)).contains(x)
        }
        divisor_list(p.pow(k)).contains(x) =
            map((k + Nat.1).range, p.pow).contains(x)
    }
}

/// `sigma(p^k)` is the geometric series `p^0 + p^1 + ... + p^k`.
theorem nat_sigma_prime_pow_sum(p: Nat, k: Nat) {
    p.is_prime implies nat_sigma(p.pow(k)) = partial(p.pow, k + Nat.1)
} by {
    if p.is_prime {
        divisor_list_is_unique(p.pow(k))
        divisor_list(p.pow(k)).is_unique
        pow_range_list_unique(p, k)
        map((k + Nat.1).range, p.pow).is_unique
        forall(x: Nat) {
            divisor_list_prime_pow_contains_iff(p, k, x)
            divisor_list(p.pow(k)).contains(x) =
                map((k + Nat.1).range, p.pow).contains(x)
        }
        unique_same_contains_map_sum_eq(
            divisor_list(p.pow(k)), map((k + Nat.1).range, p.pow),
            nat_identity_arithmetic_fn)
        sum(map(divisor_list(p.pow(k)), nat_identity_arithmetic_fn)) =
            sum(map(map((k + Nat.1).range, p.pow), nat_identity_arithmetic_fn))
        sum_map_nat_identity_arithmetic_fn_eq_sum(divisor_list(p.pow(k)))
        sum(map(divisor_list(p.pow(k)), nat_identity_arithmetic_fn)) =
            sum(divisor_list(p.pow(k)))
        sum_map_nat_identity_arithmetic_fn_eq_sum(map((k + Nat.1).range, p.pow))
        sum(map(map((k + Nat.1).range, p.pow), nat_identity_arithmetic_fn)) =
            sum(map((k + Nat.1).range, p.pow))
        nat_sigma(p.pow(k)) = sum(divisor_list(p.pow(k)))
        nat_sigma(p.pow(k)) = sum(map((k + Nat.1).range, p.pow))
        partial(p.pow, k + Nat.1) = sum(map((k + Nat.1).range, p.pow))
        nat_sigma(p.pow(k)) = partial(p.pow, k + Nat.1)
    }
}

/// The finite geometric-series identity in the naturals:
/// `(p - 1) * (p^0 + ... + p^(k-1)) + 1 = p^k`.
theorem nat_pow_sum_geometric(p: Nat, k: Nat) {
    p.is_prime implies
        (p - Nat.1) * partial(p.pow, k) + Nat.1 = p.pow(k)
} by {
    if p.is_prime {
        Nat.1 < p
        let f: Nat -> Bool = function(x: Nat) {
            (p - Nat.1) * partial(p.pow, x) + Nat.1 = p.pow(x)
        }
        partial(p.pow, Nat.0) = sum(map(Nat.0.range, p.pow))
        map(Nat.0.range, p.pow) = map(List.nil[Nat], p.pow)
        map(List.nil[Nat], p.pow) = List.nil[Nat]
        sum(List.nil[Nat]) = Nat.0
        partial(p.pow, Nat.0) = Nat.0
        (p - Nat.1) * Nat.0 = Nat.0
        Nat.0 + Nat.1 = Nat.1
        exp_zero(p)
        p.pow(Nat.0) = Nat.1
        f(Nat.0)
        forall(x: Nat) {
            if f(x) {
                partial_split_last(p.pow, x)
                partial(p.pow, x.suc) = partial(p.pow, x) + p.pow(x)
                (p - Nat.1) * (partial(p.pow, x) + p.pow(x)) =
                    (p - Nat.1) * partial(p.pow, x) +
                        (p - Nat.1) * p.pow(x)
                (p - Nat.1) * partial(p.pow, x) + Nat.1 = p.pow(x)
                (p - Nat.1) * partial(p.pow, x) +
                    (p - Nat.1) * p.pow(x) + Nat.1 =
                    (p - Nat.1) * partial(p.pow, x) +
                        ((p - Nat.1) * p.pow(x) + Nat.1)
                add_assoc((p - Nat.1) * partial(p.pow, x),
                    (p - Nat.1) * p.pow(x), Nat.1)
                (p - Nat.1) * p.pow(x) + Nat.1 =
                    Nat.1 + (p - Nat.1) * p.pow(x)
                add_comm((p - Nat.1) * p.pow(x), Nat.1)
                (p - Nat.1) * partial(p.pow, x) +
                    (Nat.1 + (p - Nat.1) * p.pow(x)) =
                    (p - Nat.1) * partial(p.pow, x) + Nat.1 +
                        (p - Nat.1) * p.pow(x)
                add_assoc((p - Nat.1) * partial(p.pow, x), Nat.1,
                    (p - Nat.1) * p.pow(x))
                (p - Nat.1) * partial(p.pow, x) + Nat.1 +
                    (p - Nat.1) * p.pow(x) =
                    p.pow(x) + (p - Nat.1) * p.pow(x)
                (p - Nat.1) * p.pow(x) = p.pow(x) * (p - Nat.1)
                p.pow(x) + (p - Nat.1) * p.pow(x) =
                    p.pow(x) + p.pow(x) * (p - Nat.1)
                p.pow(x) + p.pow(x) * (p - Nat.1) =
                    p.pow(x) * (Nat.1 + (p - Nat.1))
                sub_one_add_one(p)
                (p - Nat.1) + Nat.1 = p
                Nat.1 + (p - Nat.1) = (p - Nat.1) + Nat.1
                Nat.1 + (p - Nat.1) = p
                p.pow(x) * p = p.pow(x + Nat.1)
                exp_add(p, x, Nat.1)
                exp_one(p)
                p.pow(x + Nat.1) = p.pow(x.suc)
                x + Nat.1 = x.suc
                (p - Nat.1) * partial(p.pow, x.suc) + Nat.1 =
                    p.pow(x.suc)
                f(x.suc)
            }
        }
        f(Nat.0) and forall(x: Nat) { f(x) implies f(x.suc) }
        Nat.induction(f)
        f(k)
        (p - Nat.1) * partial(p.pow, k) + Nat.1 = p.pow(k)
    }
}

/// `sigma(p^k)` satisfies the geometric-series relation
/// `(p - 1) * sigma(p^k) + 1 = p^(k+1)`.
theorem nat_sigma_prime_pow_mult(p: Nat, k: Nat) {
    p.is_prime implies
        (p - Nat.1) * nat_sigma(p.pow(k)) + Nat.1 = p.pow(k + Nat.1)
} by {
    if p.is_prime {
        nat_sigma_prime_pow_sum(p, k)
        nat_sigma(p.pow(k)) = partial(p.pow, k + Nat.1)
        nat_pow_sum_geometric(p, k + Nat.1)
        (p - Nat.1) * partial(p.pow, k + Nat.1) + Nat.1 =
            p.pow(k + Nat.1)
        (p - Nat.1) * nat_sigma(p.pow(k)) + Nat.1 = p.pow(k + Nat.1)
    }
}

/// `sigma(p^k) = (p^(k+1) - 1) / (p - 1)` for a prime `p`.
theorem nat_sigma_prime_pow(p: Nat, k: Nat) {
    p.is_prime implies
        nat_sigma(p.pow(k)) = (p.pow(k + Nat.1) - Nat.1).div(p - Nat.1)
} by {
    if p.is_prime {
        nat_sigma_prime_pow_mult(p, k)
        (p - Nat.1) * nat_sigma(p.pow(k)) + Nat.1 = p.pow(k + Nat.1)
        add_imp_sub((p - Nat.1) * nat_sigma(p.pow(k)), Nat.1, p.pow(k + Nat.1))
        p.pow(k + Nat.1) - Nat.1 = (p - Nat.1) * nat_sigma(p.pow(k))
        Nat.1 < p
        sub_pos(p, Nat.1)
        p - Nat.1 > Nat.0
        p - Nat.1 != Nat.0
        div_mul(nat_sigma(p.pow(k)), p - Nat.1)
        (nat_sigma(p.pow(k)) * (p - Nat.1)).div(p - Nat.1) =
            nat_sigma(p.pow(k))
        nat_sigma(p.pow(k)) * (p - Nat.1) =
            (p - Nat.1) * nat_sigma(p.pow(k))
        (p.pow(k + Nat.1) - Nat.1).div(p - Nat.1) =
            nat_sigma(p.pow(k))
        nat_sigma(p.pow(k)) = (p.pow(k + Nat.1) - Nat.1).div(p - Nat.1)
    }
}

/// The divisor sum of `n` splits off the divisor `n` itself: the remaining
/// sum ranges over the other positive divisors of `n`.
theorem nat_sigma_decompose_self(n: Nat) {
    Nat.0 < n implies nat_sigma(n) = n + sum(divisor_list(n).remove_one(n))
} by {
    if Nat.0 < n {
        divides_self(n)
        n.divides(n)
        divisor_list_contains_of(n, n)
        divisor_list(n).contains(n)
        sum_map_remove_one(divisor_list(n), n, nat_identity_arithmetic_fn)
        nat_identity_arithmetic_fn(n) +
            sum(map(divisor_list(n).remove_one(n), nat_identity_arithmetic_fn)) =
            sum(map(divisor_list(n), nat_identity_arithmetic_fn))
        nat_identity_arithmetic_fn_apply(n)
        nat_identity_arithmetic_fn(n) = n
        sum_map_nat_identity_arithmetic_fn_eq_sum(
            divisor_list(n).remove_one(n))
        sum(map(divisor_list(n).remove_one(n), nat_identity_arithmetic_fn)) =
            sum(divisor_list(n).remove_one(n))
        sum_map_nat_identity_arithmetic_fn_eq_sum(divisor_list(n))
        sum(map(divisor_list(n), nat_identity_arithmetic_fn)) =
            sum(divisor_list(n))
        n + sum(divisor_list(n).remove_one(n)) = sum(divisor_list(n))
        nat_sigma(n) = sum(divisor_list(n))
        nat_sigma(n) = n + sum(divisor_list(n).remove_one(n))
    }
}

/// A list sum is at least any of its members.
theorem nat_sum_contains_ge(item: Nat, l: List[Nat]) {
    l.contains(item) implies item <= sum(l)
} by {
    if l.contains(item) {
        sum_map_remove_one(l, item, nat_identity_arithmetic_fn)
        nat_identity_arithmetic_fn(item) +
            sum(map(l.remove_one(item), nat_identity_arithmetic_fn)) =
            sum(map(l, nat_identity_arithmetic_fn))
        nat_identity_arithmetic_fn_apply(item)
        nat_identity_arithmetic_fn(item) = item
        sum_map_nat_identity_arithmetic_fn_eq_sum(l.remove_one(item))
        sum(map(l.remove_one(item), nat_identity_arithmetic_fn)) =
            sum(l.remove_one(item))
        sum_map_nat_identity_arithmetic_fn_eq_sum(l)
        sum(map(l, nat_identity_arithmetic_fn)) = sum(l)
        item + sum(l.remove_one(item)) = sum(l)
        lte_add_left(item, Nat.0, sum(l.remove_one(item)))
        item + Nat.0 <= item + sum(l.remove_one(item))
        item + Nat.0 = item
        item <= sum(l)
    }
}

/// For a proper divisor `k` of `n`, the divisor sum of `n` is at least `n + k`.
theorem nat_sigma_ge_self_plus_divisor(n: Nat, k: Nat) {
    Nat.0 < n and k.divides(n) and k != n implies n + k <= nat_sigma(n)
} by {
    if Nat.0 < n and k.divides(n) and k != n {
        nat_sigma_decompose_self(n)
        nat_sigma(n) = n + sum(divisor_list(n).remove_one(n))
        if k = Nat.0 {
            let c: Nat satisfy { k * c = n }
            Nat.0 * c = n
            Nat.0 * c = Nat.0
            n = Nat.0
            Nat.0 < n
            false
        }
        k != Nat.0
        pos_of_ne_zero(k)
        Nat.0 < k
        divisor_list_contains_of(n, k)
        divisor_list(n).contains(k)
        remove_one_contains_other(divisor_list(n), n, k)
        k != n
        divisor_list(n).contains(k) =
            divisor_list(n).remove_one(n).contains(k)
        divisor_list(n).remove_one(n).contains(k)
        nat_sum_contains_ge(k, divisor_list(n).remove_one(n))
        k <= sum(divisor_list(n).remove_one(n))
        lte_add_left(n, k, sum(divisor_list(n).remove_one(n)))
        n + k <= n + sum(divisor_list(n).remove_one(n))
        n + sum(divisor_list(n).remove_one(n)) = nat_sigma(n)
        n + k <= nat_sigma(n)
    }
}

/// `sigma(n) >= n + 1` for every `n > 1`.
theorem nat_sigma_ge_self_plus_one(n: Nat) {
    Nat.1 < n implies n + Nat.1 <= nat_sigma(n)
} by {
    if Nat.1 < n {
        one_divides_nat(n)
        Nat.1.divides(n)
        Nat.1 != n
        lt_trans(Nat.0, Nat.1, n)
        Nat.0 < n
        nat_sigma_ge_self_plus_divisor(n, Nat.1)
        n + Nat.1 <= nat_sigma(n)
    }
}

/// `sigma(n) = n + 1` exactly when `n` is prime.
theorem nat_sigma_eq_self_plus_one_iff_prime(n: Nat) {
    (nat_sigma(n) = n + Nat.1) = n.is_prime
} by {
    if n.is_prime {
        nat_sigma_prime(n)
        nat_sigma(n) = n + Nat.1
    }
    if nat_sigma(n) = n + Nat.1 {
        if n = Nat.0 {
            nat_sigma_zero
            nat_sigma(Nat.0) = Nat.0
            nat_sigma(n) = n + Nat.1
            Nat.0 = Nat.0 + Nat.1
            Nat.0 + Nat.1 = Nat.1
            Nat.0 = Nat.1
            false
        }
        if n = Nat.1 {
            nat_sigma_one
            nat_sigma(Nat.1) = Nat.1
            nat_sigma(n) = n + Nat.1
            Nat.1 = Nat.1 + Nat.1
            Nat.1 + Nat.1 = Nat.2
            Nat.1 = Nat.2
            false
        }
        n != Nat.0
        pos_of_ne_zero(n)
        Nat.0 < n
        lt_imp_lte_suc(Nat.0, n)
        Nat.1 <= n
        n != Nat.1
        trichotomy(Nat.1, n)
        if n < Nat.1 {
            lt_suc_right(n, Nat.0)
            n = Nat.0
            false
        }
        if Nat.1 = n {
            false
        }
        Nat.1 < n
        forall(k: Nat) {
            if Nat.1 < k and k < n {
                if k.divides(n) {
                    k != n
                    Nat.0 < n
                    nat_sigma_ge_self_plus_divisor(n, k)
                    n + k <= nat_sigma(n)
                    n + k <= n + Nat.1
                    lt_imp_lte_suc(Nat.1, k)
                    Nat.2 <= k
                    lte_add_left(n, Nat.2, k)
                    n + Nat.2 <= n + k
                    lte_trans(n + Nat.2, n + k, n + Nat.1)
                    n + Nat.2 <= n + Nat.1
                    lt_suc(n + Nat.1)
                    n + Nat.1 < (n + Nat.1).suc
                    (n + Nat.1).suc = n + Nat.2
                    n + Nat.1 < n + Nat.2
                    lte_imp_not_lt(n + Nat.2, n + Nat.1)
                    false
                }
                not k.divides(n)
            }
        }
        no_proper_divisor_imp_prime(n)
        n.is_prime
    }
    (nat_sigma(n) = n + Nat.1) = n.is_prime
}

/// `sigma` is multiplicative on positive coprime arguments:
/// `sigma(m * n) = sigma(m) * sigma(n)` whenever `gcd(m, n) = 1`.
/// The proof lives in `number_theory.dirichlet` via Dirichlet convolution
/// `sigma = id * 1`; this file re-exports the statement, and the full
/// `is_multiplicative_nat_fn(nat_sigma)` theorem is
/// `nat_sigma_multiplicative` in `number_theory.dirichlet`.
theorem nat_sigma_mul_coprime(m: Nat, n: Nat) {
    Nat.0 < m and Nat.0 < n and m.coprime(n) implies
        nat_sigma(m * n) = nat_sigma(m) * nat_sigma(n)
} by {
    if Nat.0 < m and Nat.0 < n and m.coprime(n) {
        nat_sigma_mul_coprime_positive(m, n)
        nat_sigma(m * n) = nat_sigma(m) * nat_sigma(n)
    }
}
