/// The counting mechanism of the 2026 disproof of the Erdős unit-distance
/// conjecture.
///
/// Alon–Bloom–Gowers–Litt–Sawin–Shankar–Tsimerman–Wang–Wood (arXiv
/// 2605.20695) disproved the Erdős unit-distance conjecture by constructing,
/// for a fixed eps > 0, arbitrarily large planar point sets with more than
/// n^(1 + eps) unit distances.  The engine of the construction is a counting
/// mechanism that this file isolates and proves:  if a finite point set `s`
/// contains a window `interior` whose translates by every vector of a finite
/// set `u` of unit-norm vectors stay inside `s`, then the translates are
/// |interior|·|u| distinct ordered unit pairs, i.e.
///
///     fs_card(ordered_unit_pairs(s)) >= |interior| * |u|
///
/// (`translate_count_core`).  This is the abstract content of Lemma 2.1 of
/// the paper:  with a lattice window Λ ∩ B_R of a number-field lattice and
/// the unit-norm elements U_Λ, every lattice point of the smaller window
/// B_{R−1} translated by any element of U_Λ stays in B_R, which yields
/// 2·ν(𝒫) ≥ |U_Λ|·|Λ ∩ B_{R−1}| for the projected point set 𝒫.
///
/// The four-dimensional grid of `number_theory/quaternion_grid_count.ac` is
/// the constant-|U| case:  |u| = 8 (the unit vectors ±eᵢ), so the mechanism
/// gives the linear lower bound 2·ν ≥ 8·(n − 2)⁴ from the (n − 2)⁴ interior
/// points.  The Golod–Shafarevich construction of the paper is the growing
/// case:  along a tower of CM fields of bounded root discriminant, the set
/// U_Λ of unit-norm algebraic numbers of bounded denominator has
/// |U_Λ| ≥ u^f with the degree f → ∞ while the lattice "skewness" v and the
/// minimal coordinate size δ stay bounded; Lemma 2.1 then gives
///
///     2·ν(𝒫) ≥ (u·π·R² / (4·v·δ²))^f    and    |𝒫| ≤ (9·R² / δ²)^f,
///
/// and any u > 36·v/π forces (2ν).log/log|𝒫| > 1 uniformly in f, i.e.
/// |𝒫|^(1 + eps) unit distances for a fixed eps > 0 (the commented theorem
/// `erdos_unit_distance_superlinear_2026` below).  The two lemmas of the
/// paper that feed the mechanism — Lemma 2.1 (geometry of numbers) and
/// Lemma 2.2 (the pigeonhole count |U| ≥ ∏(k_j + 1)/h(K) of unit-norm
/// algebraic numbers of bounded denominator) — are recorded as commented
/// statements after the proved core.
from nat import Nat
from real import Real
from pair import Pair, pair_new_first, pair_new_second, pair_ext
from finite_set import FiniteSet, fs_image, finite_set_image_contains_eq,
    finite_set_image_cardinality_is_of_injective, finite_set_subset_contains
from data.basic.functions import is_injective_fn
from data.finite.finite_set_card import fs_card, fs_card_eq_of_cardinality_is, fs_card_cardinality_is
from data.finite.finite_set_filter import finite_set_filter, finite_set_filter_contains_eq
from data.finite.finite_set_product import finite_set_product, finite_set_product_contains_eq,
    finite_set_product_contains_pair
from data.finite.finite_set_card_compare import fs_card_mono
from data.finite.finite_set_product_card import fs_card_product
from data.finite.finite_set_subset_intro import fs_subset_eq_intro
from number_theory.quaternion_unit_distance import Quat, quat_add, quat_norm,
    ordered_unit_pairs, ordered_unit_pair, quaternion_unit_distance, quat_dist_sq,
    quat_dist_sq_add_right, quat_add_left_cancel

numerals Real
numerals Nat

/// The ordered unit pair obtained from a base point and a unit step:
/// (a, a + w).
define step_pair_of(a: Quat, w: Quat) -> Pair[Quat, Quat] {
    Pair.new(a, quat_add(a, w))
}

/// The map (a, w) ↦ (a, a + w) on pairs of quaternions.
define pair_step_fn(p: Pair[Quat, Quat]) -> Pair[Quat, Quat] {
    step_pair_of(p.first, p.second)
}

/// A translate by a unit vector is a unit-distance pair.
theorem step_pair_unit_dist(a: Quat, w: Quat) {
    quat_norm(w) = Real.1 implies quaternion_unit_distance(a, quat_add(a, w))
} by {
    if quat_norm(w) = Real.1 {
        quaternion_unit_distance(a, quat_add(a, w)) =
            (quat_dist_sq(a, quat_add(a, w)) = Real.1)
        quat_dist_sq_add_right(a, w)
        quat_dist_sq(a, quat_add(a, w)) = quat_norm(w)
        quat_dist_sq(a, quat_add(a, w)) = Real.1
        quaternion_unit_distance(a, quat_add(a, w))
    }
}

/// The step map (a, w) ↦ (a, a + w) is injective.
theorem pair_step_fn_injective {
    is_injective_fn(pair_step_fn)
} by {
    forall(p: Pair[Quat, Quat], q: Pair[Quat, Quat]) {
        if pair_step_fn(p) = pair_step_fn(q) {
            pair_step_fn(p) = Pair.new(p.first, quat_add(p.first, p.second))
            pair_step_fn(q) = Pair.new(q.first, quat_add(q.first, q.second))
            Pair.new(p.first, quat_add(p.first, p.second)) =
                Pair.new(q.first, quat_add(q.first, q.second))
            pair_new_first(p.first, quat_add(p.first, p.second))
            Pair.new(p.first, quat_add(p.first, p.second)).first = p.first
            pair_new_first(q.first, quat_add(q.first, q.second))
            Pair.new(q.first, quat_add(q.first, q.second)).first = q.first
            p.first = q.first
            pair_new_second(p.first, quat_add(p.first, p.second))
            Pair.new(p.first, quat_add(p.first, p.second)).second = quat_add(p.first, p.second)
            pair_new_second(q.first, quat_add(q.first, q.second))
            Pair.new(q.first, quat_add(q.first, q.second)).second = quat_add(q.first, q.second)
            Pair.new(p.first, quat_add(p.first, p.second)) =
                Pair.new(q.first, quat_add(q.first, q.second))
            quat_add(p.first, p.second) = quat_add(q.first, q.second)
            p.first = q.first
            quat_add(p.first, p.second) = quat_add(p.first, q.second)
            quat_add_left_cancel(p.first, p.second, q.second)
            p.second = q.second
            pair_ext(p, q)
            p.first = q.first and p.second = q.second
            p = q
        }
        pair_step_fn(p) = pair_step_fn(q) implies p = q
    }
    is_injective_fn(pair_step_fn) = forall(p: Pair[Quat, Quat], q: Pair[Quat, Quat]) {
        pair_step_fn(p) = pair_step_fn(q) implies p = q
    }
    is_injective_fn(pair_step_fn)
}

/// A translated interior point is an ordered unit pair of the containing set.
theorem step_pair_member_oup(s: FiniteSet[Quat], interior: FiniteSet[Quat], u: FiniteSet[Quat],
    p: Pair[Quat, Quat]) {
    interior.subset_eq(s) and
    (forall(q: Pair[Quat, Quat]) {
        finite_set_product(interior, u).contains(q) implies s.contains(quat_add(q.first, q.second))
    }) and
    (forall(w: Quat) {
        u.contains(w) implies quat_norm(w) = Real.1
    }) and
    finite_set_product(interior, u).contains(p)
    implies ordered_unit_pairs(s).contains(pair_step_fn(p))
} by {
    if interior.subset_eq(s) and
        (forall(q: Pair[Quat, Quat]) {
            finite_set_product(interior, u).contains(q) implies s.contains(quat_add(q.first, q.second))
        }) and
        (forall(w: Quat) {
            u.contains(w) implies quat_norm(w) = Real.1
        }) and
        finite_set_product(interior, u).contains(p) {
        finite_set_product_contains_eq(interior, u, p)
        finite_set_product(interior, u).contains(p) =
            (interior.contains(p.first) and u.contains(p.second))
        interior.contains(p.first) and u.contains(p.second)
        interior.contains(p.first)
        u.contains(p.second)
        finite_set_subset_contains(interior, s, p.first)
        interior.subset_eq(s) and interior.contains(p.first)
        s.contains(p.first)
        forall(q: Pair[Quat, Quat]) {
            finite_set_product(interior, u).contains(q) implies s.contains(quat_add(q.first, q.second))
        }
        forall(w: Quat) {
            u.contains(w) implies quat_norm(w) = Real.1
        }
        s.contains(quat_add(p.first, p.second))
        quat_norm(p.second) = Real.1
        step_pair_unit_dist(p.first, p.second)
        quaternion_unit_distance(p.first, quat_add(p.first, p.second))
        pair_step_fn(p) = Pair.new(p.first, quat_add(p.first, p.second))
        finite_set_product_contains_pair(s, s, p.first, quat_add(p.first, p.second))
        s.contains(p.first) and s.contains(quat_add(p.first, p.second))
        finite_set_product(s, s).contains(Pair.new(p.first, quat_add(p.first, p.second)))
        ordered_unit_pair(Pair.new(p.first, quat_add(p.first, p.second)))
        finite_set_filter_contains_eq(finite_set_product(s, s), ordered_unit_pair,
            Pair.new(p.first, quat_add(p.first, p.second)))
        finite_set_filter(finite_set_product(s, s), ordered_unit_pair).contains(
            Pair.new(p.first, quat_add(p.first, p.second))) =
            (finite_set_product(s, s).contains(Pair.new(p.first, quat_add(p.first, p.second))) and
                ordered_unit_pair(Pair.new(p.first, quat_add(p.first, p.second))))
        finite_set_filter(finite_set_product(s, s), ordered_unit_pair).contains(
            Pair.new(p.first, quat_add(p.first, p.second)))
        ordered_unit_pairs(s) = finite_set_filter(finite_set_product(s, s), ordered_unit_pair)
        ordered_unit_pairs(s).contains(Pair.new(p.first, quat_add(p.first, p.second)))
        pair_step_fn(p) = Pair.new(p.first, quat_add(p.first, p.second))
        ordered_unit_pairs(s).contains(pair_step_fn(p))
    }
}

/// The counting core of the 2026 disproof:  every unit step of a window
/// gives a distinct ordered unit pair.
///
/// If every point of `interior` stays inside `s` when translated by any
/// vector of the unit-norm set `u`, then the |interior|·|u| translates are
/// distinct ordered unit pairs of `s`.
theorem translate_count_core(s: FiniteSet[Quat], interior: FiniteSet[Quat], u: FiniteSet[Quat]) {
    interior.subset_eq(s) and
    (forall(q: Pair[Quat, Quat]) {
        finite_set_product(interior, u).contains(q) implies s.contains(quat_add(q.first, q.second))
    }) and
    (forall(w: Quat) {
        u.contains(w) implies quat_norm(w) = Real.1
    })
    implies fs_card(ordered_unit_pairs(s)) >= fs_card(interior) * fs_card(u)
} by {
    if interior.subset_eq(s) and
        (forall(q: Pair[Quat, Quat]) {
            finite_set_product(interior, u).contains(q) implies s.contains(quat_add(q.first, q.second))
        }) and
        (forall(w: Quat) {
            u.contains(w) implies quat_norm(w) = Real.1
        }) {
        finite_set_product(interior, u).cardinality_is(
            fs_card(finite_set_product(interior, u)))
        pair_step_fn_injective
        is_injective_fn(pair_step_fn)
        finite_set_image_cardinality_is_of_injective(finite_set_product(interior, u),
            pair_step_fn, fs_card(finite_set_product(interior, u)))
        is_injective_fn(pair_step_fn) and
            finite_set_product(interior, u).cardinality_is(
                fs_card(finite_set_product(interior, u)))
        fs_image(finite_set_product(interior, u), pair_step_fn).cardinality_is(
            fs_card(finite_set_product(interior, u)))
        fs_card_eq_of_cardinality_is(fs_image(finite_set_product(interior, u), pair_step_fn),
            fs_card(finite_set_product(interior, u)))
        fs_card(fs_image(finite_set_product(interior, u), pair_step_fn)) =
            fs_card(finite_set_product(interior, u))
        fs_card_product(interior, u)
        fs_card(finite_set_product(interior, u)) = fs_card(u) * fs_card(interior)
        fs_card(u) * fs_card(interior) = fs_card(interior) * fs_card(u)
        fs_card(fs_image(finite_set_product(interior, u), pair_step_fn)) =
            fs_card(interior) * fs_card(u)
        forall(p: Pair[Quat, Quat]) {
            if fs_image(finite_set_product(interior, u), pair_step_fn).contains(p) {
                finite_set_image_contains_eq(finite_set_product(interior, u), pair_step_fn, p)
                fs_image(finite_set_product(interior, u), pair_step_fn).contains(p) =
                    exists(q: Pair[Quat, Quat]) {
                        finite_set_product(interior, u).contains(q) and p = pair_step_fn(q)
                    }
                exists(q: Pair[Quat, Quat]) {
                    finite_set_product(interior, u).contains(q) and p = pair_step_fn(q)
                }
                let (q: Pair[Quat, Quat]) satisfy {
                    finite_set_product(interior, u).contains(q) and p = pair_step_fn(q)
                }
                finite_set_product(interior, u).contains(q)
                p = pair_step_fn(q)
                step_pair_member_oup(s, interior, u, q)
                interior.subset_eq(s) and
                    (forall(r: Pair[Quat, Quat]) {
                        finite_set_product(interior, u).contains(r) implies s.contains(quat_add(r.first, r.second))
                    }) and
                    (forall(w: Quat) {
                        u.contains(w) implies quat_norm(w) = Real.1
                    }) and
                    finite_set_product(interior, u).contains(q)
                ordered_unit_pairs(s).contains(pair_step_fn(q))
                p = pair_step_fn(q)
                ordered_unit_pairs(s).contains(p)
            }
            fs_image(finite_set_product(interior, u), pair_step_fn).contains(p) implies ordered_unit_pairs(s).contains(p)
        }
        fs_subset_eq_intro(fs_image(finite_set_product(interior, u), pair_step_fn),
            ordered_unit_pairs(s))
        fs_image(finite_set_product(interior, u), pair_step_fn).subset_eq(
            ordered_unit_pairs(s))
        fs_card_mono(fs_image(finite_set_product(interior, u), pair_step_fn),
            ordered_unit_pairs(s))
        fs_card(fs_image(finite_set_product(interior, u), pair_step_fn)) <= fs_card(ordered_unit_pairs(s))
        fs_card(ordered_unit_pairs(s)) >= fs_card(fs_image(finite_set_product(interior, u), pair_step_fn))
        fs_card(ordered_unit_pairs(s)) >= fs_card(interior) * fs_card(u)
    }
}

// ============================================================================
// The grid as the constant-|U| case, and the Golod-Shafarevich framework
// ============================================================================
//
// The four-dimensional grid of `number_theory/quaternion_grid_count.ac` is
// the case |u| = 8:  the window is the n by n by n by n grid, the interior
// is its (n - 2)⁴ inner points, and `u` is the set of the eight unit vectors
// ±e₁, ±e₂, ±e₃, ±e₄, each of norm one (proved there).  The core then gives
// the linear lower bound
//
//     fs_card(ordered_unit_pairs(grid(n))) >= 8 * (n - 2)^4,
//
// i.e. 2·ν ≥ 8·(n − 2)⁴ — the classical grid side of the problem.  The 2026
// disproof needs the growing case, in which |u| itself grows:  Lemma 2.1 of
// the paper converts a lattice with many unit-norm elements into a planar
// point set, and the Golod–Shafarevich class field towers provide the
// lattices.  The two lemmas and the final statement are recorded here:
//
//   // Lemma 2.1 (geometry of numbers) of arXiv 2605.20695.  For a lattice
//   // Λ ⊂ ℂ^f with covolume covol(Λ), minimal coordinate size δ and
//   // |U_Λ| ≥ u^f unit-norm elements, there is a translate a + Λ whose
//   // window points, projected to a coordinate, form a planar set 𝒫 with
//   //   2·ν(𝒫) ≥ (u·π·R²/(4·v·δ²))^f  and  |𝒫| ≤ (9·R²/δ²)^f,
//   // where v ≥ δ⁻²·covol(Λ)^(1/f) bounds the skewness of the lattice.
//   theorem lattice_window_superlinear_count(...) { ... }
//
//   // Lemma 2.2 (pigeonhole count of unit-norm algebraic numbers).  For a
//   // CM field K with class number h(K) and prime ideals P₁, ..., Pₛ with
//   // Q = ∏(P_j·conj(P_j))^{k_j}, the set U = { u ∈ Q⁻² : |u| = 1 } of
//   // unit-norm algebraic numbers of bounded denominator satisfies
//   //   |U| ≥ ∏(k_j + 1)/h(K),
//   // and Q⁻² ⊆ D⁻¹·O_K for the explicit integer D.  Taking the P_j above a
//   // fixed split rational prime and k_j large against the bounded root
//   // discriminant of a Golod–Shafarevich tower gives |U_Λ| ≥ u^f with
//   // u > 36·v/π.
//   theorem unit_norm_elements_pigeonhole(...) { ... }
//
//   // The final statement:  for a fixed eps > 0 there are arbitrarily large
//   // point sets in the plane with more than n^(1 + eps) unit distances.
//   // (This records the 2026 disproof; the Golod-Shafarevich construction
//   // itself is not formalized here.)
//   theorem erdos_unit_distance_superlinear_2026 {
//       exists(eps: Real) {
//           Real.0 < eps and forall(n0: Nat) {
//               exists(n: Nat) {
//                   n0 <= n and exists(s: FiniteSet[Quat]) {
//                       s.cardinality = n and
//                       from_nat[Real](n).pow(Real.1 + eps) <
//                           from_nat[Real](Nat.2 * nu(s))
//                   }
//               }
//           }
//       }
//   }
//
// The mechanism that turns the lemmas into the exponent is the ratio
// argument:  from 2·ν(𝒫) ≥ (u·π·R²/(4·v·δ²))^f and |𝒫| ≤ (9·R²/δ²)^f with
// u > 36·v/π, both bounds are powers of R² with bases b₁ > b₂ > 1, so
//
//     (2·ν(𝒫)).log / log|𝒫| ≥ (b₁).log/(b₂).log > 1
//
// uniformly in f, and with |𝒫| → ∞ the ratio stays above 1 + eps for a
// fixed eps > 0.  This is exactly the "counting mechanism":  many unit-norm
// vectors (many w in `u`) times many window points (many a in `interior`)
// produces many unit distances.
