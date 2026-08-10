/// Chinese remainder theorem applications.
///
/// This file collects the classical applications of the CRT machinery: the
/// simultaneous-congruence example `x ≡ 2 (mod 3), x ≡ 3 (mod 5)` with its
/// unique solution `8` modulo 15, the general unique-solution statement for
/// pairwise-coprime systems of any length, the multiplicativity of Euler's
/// totient `φ(mn) = φ(m)φ(n)` for coprime `m`, `n` through the CRT pairing,
/// and the structure of `Z/(mn)` as `Z/m × Z/n`.
///
/// The existence, uniqueness, and canonical-solution machinery lives in
/// `crt.ac`, `crt_list.ac`, and `crt_canonical.ac`; this file instantiates
/// it on concrete systems and restates the classical consequences.

from nat import Nat, small_mod, divides_self, divides_mul, mul_to_zero, mod_lt,
    add_suc_right, add_zero_right, lt_add_suc, mod_of_decomp
from number_theory.congruence import congr_mod_symm, congr_mod_trans,
    mod_congr_mod_self
from number_theory.coprime import coprime_mul, coprime_mul_iff, coprime_mod_iff,
    coprime_mod_imp
from number_theory.crt import nat_crt_two_moduli, nat_congr_combine_coprime
from number_theory.crt_list import satisfies_all, system_moduli, system_modulus,
    every_modulus_positive, nat_crt_list, satisfies_all_unique_mod_system_modulus,
    satisfies_all_nil, satisfies_all_cons_intro, system_modulus_cons,
    nat_congr_mod_descend
from number_theory.crt_canonical import crt_canonical_solution,
    normalized_solution_eq_crt_canonical
from number_theory.pairwise_coprime import pairwise_coprime, coprime_with_all,
    pairwise_coprime_cons
from number_theory.carmichael import three_is_prime
from number_theory.zsigmondy import five_is_prime
from number_theory.factorisation import coprime_of_distinct_primes
from number_theory.totient import nat_totient, nat_totient_mul_coprime
from list import List
from pair import Pair
numerals Nat

// ---------------------------------------------------------------------------
// Small arithmetic facts used by the concrete example.
// ---------------------------------------------------------------------------

/// `8 + 1 = 9`.
theorem crt_example_8_add_1 {
    Nat.8 + Nat.1 = Nat.9
} by {
    add_suc_right(Nat.8, Nat.0)
    Nat.8 + Nat.1 = (Nat.8 + Nat.0).suc
    add_zero_right(Nat.8)
    Nat.8 + Nat.0 = Nat.8
    (Nat.8 + Nat.0).suc = Nat.8.suc
    Nat.8.suc = Nat.9
    Nat.8 + Nat.1 = Nat.9
}

/// `8 + 2 = 10`.
theorem crt_example_8_add_2 {
    Nat.8 + Nat.2 = Nat.10
} by {
    add_suc_right(Nat.8, Nat.1)
    Nat.8 + Nat.2 = (Nat.8 + Nat.1).suc
    crt_example_8_add_1
    Nat.8 + Nat.1 = Nat.9
    (Nat.8 + Nat.1).suc = Nat.9.suc
    Nat.9.suc = Nat.10
    Nat.8 + Nat.2 = Nat.10
}

/// `8 + 3 = 11`.
theorem crt_example_8_add_3 {
    Nat.8 + Nat.3 = Nat.11
} by {
    add_suc_right(Nat.8, Nat.2)
    Nat.8 + Nat.3 = (Nat.8 + Nat.2).suc
    crt_example_8_add_2
    Nat.8 + Nat.2 = Nat.10
    (Nat.8 + Nat.2).suc = Nat.10.suc
    Nat.10.suc = Nat.11
    Nat.8 + Nat.3 = Nat.11
}

/// `8 + 4 = 12`.
theorem crt_example_8_add_4 {
    Nat.8 + Nat.4 = Nat.12
} by {
    add_suc_right(Nat.8, Nat.3)
    Nat.8 + Nat.4 = (Nat.8 + Nat.3).suc
    crt_example_8_add_3
    Nat.8 + Nat.3 = Nat.11
    (Nat.8 + Nat.3).suc = Nat.11.suc
    Nat.11.suc = Nat.12
    Nat.8 + Nat.4 = Nat.12
}

/// `8 + 5 = 13`.
theorem crt_example_8_add_5 {
    Nat.8 + Nat.5 = Nat.13
} by {
    add_suc_right(Nat.8, Nat.4)
    Nat.8 + Nat.5 = (Nat.8 + Nat.4).suc
    crt_example_8_add_4
    Nat.8 + Nat.4 = Nat.12
    (Nat.8 + Nat.4).suc = Nat.12.suc
    Nat.12.suc = Nat.13
    Nat.8 + Nat.5 = Nat.13
}

/// `8 + 6 = 14`.
theorem crt_example_8_add_6 {
    Nat.8 + Nat.6 = Nat.14
} by {
    add_suc_right(Nat.8, Nat.5)
    Nat.8 + Nat.6 = (Nat.8 + Nat.5).suc
    crt_example_8_add_5
    Nat.8 + Nat.5 = Nat.13
    (Nat.8 + Nat.5).suc = Nat.13.suc
    Nat.13.suc = Nat.14
    Nat.8 + Nat.6 = Nat.14
}

/// `8 + 7 = 15`.
theorem crt_example_8_add_7 {
    Nat.8 + Nat.7 = Nat.15
} by {
    add_suc_right(Nat.8, Nat.6)
    Nat.8 + Nat.7 = (Nat.8 + Nat.6).suc
    crt_example_8_add_6
    Nat.8 + Nat.6 = Nat.14
    (Nat.8 + Nat.6).suc = Nat.14.suc
    Nat.14.suc = Nat.15
    Nat.8 + Nat.7 = Nat.15
}

/// `8 < 15`.
theorem crt_example_8_lt_15 {
    Nat.8 < Nat.15
} by {
    lt_add_suc(Nat.8, Nat.6)
    Nat.8 < Nat.8 + Nat.7
    crt_example_8_add_7
    Nat.8 + Nat.7 = Nat.15
    Nat.8 < Nat.15
}

/// `2 < 3`.
theorem crt_example_2_lt_3 {
    Nat.2 < Nat.3
} by {
    lt_add_suc(Nat.2, Nat.0)
    Nat.2 < Nat.2 + Nat.1
    Nat.2 + Nat.1 = Nat.3
    Nat.2 < Nat.3
}

/// `3 < 5`.
theorem crt_example_3_lt_5 {
    Nat.3 < Nat.5
} by {
    lt_add_suc(Nat.3, Nat.1)
    Nat.3 < Nat.3 + Nat.2
    Nat.3 + Nat.2 = Nat.5
    Nat.3 < Nat.5
}

/// The moduli 3 and 5 are coprime: both are distinct primes.
theorem crt_example_3_coprime_5 {
    Nat.3.coprime(Nat.5)
} by {
    three_is_prime
    Nat.3.is_prime
    five_is_prime
    Nat.5.is_prime
    Nat.3 != Nat.5
    coprime_of_distinct_primes(Nat.3, Nat.5)
    Nat.3.coprime(Nat.5)
}

// ---------------------------------------------------------------------------
// The classic example: x ≡ 2 (mod 3), x ≡ 3 (mod 5) → x ≡ 8 (mod 15).
// ---------------------------------------------------------------------------

/// The classic example system: `x ≡ 2 (mod 3)` paired with `x ≡ 3 (mod 5)`.
let crt_example_system: List[Pair[Nat, Nat]] =
    List.cons(Pair.new(Nat.3, Nat.2),
        List.cons(Pair.new(Nat.5, Nat.3), List.nil[Pair[Nat, Nat]]))

/// The classic CRT example has a simultaneous solution: instantiating the
/// two-modulus CRT at the coprime moduli 3 and 5 with residues 2 and 3.
theorem crt_example_solution_exists {
    exists(x: Nat) { x.congr_mod(Nat.2, Nat.3) and x.congr_mod(Nat.3, Nat.5) }
} by {
    crt_example_3_coprime_5
    nat_crt_two_moduli(Nat.3, Nat.5, Nat.2, Nat.3)
    Nat.3 != Nat.0
    Nat.5 != Nat.0
}

/// The classic example is solved by `x = 8`: `8 ≡ 2 (mod 3)` and
/// `8 ≡ 3 (mod 5)`, computed directly from the remainders.
theorem crt_example_eight_solves {
    Nat.8.congr_mod(Nat.2, Nat.3) and Nat.8.congr_mod(Nat.3, Nat.5)
} by {
    Nat.2 * Nat.3 + Nat.2 = Nat.8
    mod_of_decomp(Nat.2, Nat.2, Nat.3)
    crt_example_2_lt_3
    (Nat.2 * Nat.3 + Nat.2).mod(Nat.3) = Nat.2
    Nat.8.mod(Nat.3) = Nat.2
    small_mod(Nat.2, Nat.3)
    Nat.2.mod(Nat.3) = Nat.2
    Nat.8.congr_mod(Nat.2, Nat.3)
    Nat.1 * Nat.5 = Nat.5
    Nat.5 + Nat.3 = Nat.8
    Nat.1 * Nat.5 + Nat.3 = Nat.8
    mod_of_decomp(Nat.1, Nat.3, Nat.5)
    crt_example_3_lt_5
    (Nat.1 * Nat.5 + Nat.3).mod(Nat.5) = Nat.3
    Nat.8.mod(Nat.5) = Nat.3
    small_mod(Nat.3, Nat.5)
    Nat.3.mod(Nat.5) = Nat.3
    Nat.8.congr_mod(Nat.3, Nat.5)
    Nat.8.congr_mod(Nat.2, Nat.3) and Nat.8.congr_mod(Nat.3, Nat.5)
}

/// Every solution of the classic system is congruent to `8` modulo 15:
/// uniqueness of the simultaneous solution modulo the product of the
/// coprime moduli.
theorem crt_example_unique_mod_fifteen(c: Nat) {
    c.congr_mod(Nat.2, Nat.3) and c.congr_mod(Nat.3, Nat.5)
        implies c.congr_mod(Nat.8, Nat.15)
} by {
    if c.congr_mod(Nat.2, Nat.3) and c.congr_mod(Nat.3, Nat.5) {
        crt_example_eight_solves
        Nat.8.congr_mod(Nat.2, Nat.3)
        Nat.8.congr_mod(Nat.3, Nat.5)
        congr_mod_symm(Nat.8, Nat.2, Nat.3)
        Nat.2.congr_mod(Nat.8, Nat.3)
        congr_mod_trans(c, Nat.2, Nat.8, Nat.3)
        c.congr_mod(Nat.8, Nat.3)
        congr_mod_symm(Nat.8, Nat.3, Nat.5)
        Nat.3.congr_mod(Nat.8, Nat.5)
        congr_mod_trans(c, Nat.3, Nat.8, Nat.5)
        c.congr_mod(Nat.8, Nat.5)
        crt_example_3_coprime_5
        nat_congr_combine_coprime(Nat.3, Nat.5, c, Nat.8)
        c.congr_mod(Nat.8, Nat.15)
    }
}

/// The unique solution of the classic system below the combined modulus 15
/// is `8` itself.
theorem crt_example_unique_below_fifteen(c: Nat) {
    c.congr_mod(Nat.2, Nat.3) and c.congr_mod(Nat.3, Nat.5) and c < Nat.15
        implies c = Nat.8
} by {
    if c.congr_mod(Nat.2, Nat.3) and c.congr_mod(Nat.3, Nat.5) and c < Nat.15 {
        crt_example_unique_mod_fifteen(c)
        c.congr_mod(Nat.8, Nat.15)
        crt_example_8_lt_15
        small_mod(c, Nat.15)
        c.mod(Nat.15) = c
        small_mod(Nat.8, Nat.15)
        Nat.8.mod(Nat.15) = Nat.8
        c.congr_mod(Nat.8, Nat.15)
        c.mod(Nat.15) = Nat.8.mod(Nat.15)
        c = c.mod(Nat.15)
        c = Nat.8
    }
}

/// Building a system-moduli list from a cons pair: the head modulus is the
/// first projection, the tail is unchanged.
theorem crt_system_moduli_cons(head: Pair[Nat, Nat], tail: List[Pair[Nat, Nat]]) {
    system_moduli(List.cons(head, tail)) =
        List.cons(head.first, system_moduli(tail))
}

/// Coprimality with a cons list is the coprimality with its head together
/// with coprimality with its tail.
theorem crt_coprime_with_all_cons(head: Nat, t_head: Nat, t_tail: List[Nat]) {
    head.coprime(t_head) and coprime_with_all(head, t_tail) implies
        coprime_with_all(head, List.cons(t_head, t_tail))
}

/// Every natural is coprime with all of the empty list.
theorem crt_coprime_with_all_nil(head: Nat) {
    coprime_with_all(head, List.nil[Nat])
}

/// Positivity of a cons system is the positivity of its head modulus
/// together with the positivity of the tail.
theorem crt_every_modulus_positive_cons(
    head: Pair[Nat, Nat], tail: List[Pair[Nat, Nat]]
) {
    head.first != Nat.0 and every_modulus_positive(tail) implies
        every_modulus_positive(List.cons(head, tail))
}

/// The empty system has no positive moduli to check.
theorem crt_every_modulus_positive_nil {
    every_modulus_positive(List.nil[Pair[Nat, Nat]])
}

/// The moduli of the classic example system, unfolded: `[3, 5]`.
theorem crt_example_system_moduli {
    system_moduli(crt_example_system) =
        List.cons(Nat.3, List.cons(Nat.5, List.nil[Nat]))
} by {
    crt_system_moduli_cons(Pair.new(Nat.3, Nat.2),
        List.cons(Pair.new(Nat.5, Nat.3), List.nil[Pair[Nat, Nat]]))
    system_moduli(crt_example_system) =
        List.cons(Nat.3, system_moduli(List.cons(Pair.new(Nat.5, Nat.3), List.nil[Pair[Nat, Nat]])))
    crt_system_moduli_cons(Pair.new(Nat.5, Nat.3), List.nil[Pair[Nat, Nat]])
    system_moduli(List.cons(Pair.new(Nat.5, Nat.3), List.nil[Pair[Nat, Nat]])) =
        List.cons(Nat.5, system_moduli(List.nil[Pair[Nat, Nat]]))
    system_moduli(List.nil[Pair[Nat, Nat]]) = List.nil[Nat]
    system_moduli(List.cons(Pair.new(Nat.5, Nat.3), List.nil[Pair[Nat, Nat]])) =
        List.cons(Nat.5, List.nil[Nat])
    system_moduli(crt_example_system) =
        List.cons(Nat.3, List.cons(Nat.5, List.nil[Nat]))
}

/// The moduli of the classic example system are pairwise coprime:
/// `3` and `5` are distinct primes.
theorem crt_example_pairwise_coprime {
    pairwise_coprime(system_moduli(crt_example_system))
} by {
    crt_example_system_moduli
    system_moduli(crt_example_system) =
        List.cons(Nat.3, List.cons(Nat.5, List.nil[Nat]))
    crt_example_3_coprime_5
    crt_coprime_with_all_nil(Nat.3)
    crt_coprime_with_all_cons(Nat.3, Nat.5, List.nil[Nat])
    coprime_with_all(Nat.3, List.cons(Nat.5, List.nil[Nat]))
    crt_coprime_with_all_nil(Nat.5)
    pairwise_coprime(List.cons(Nat.5, List.nil[Nat]))
    pairwise_coprime_cons(Nat.3, List.cons(Nat.5, List.nil[Nat]))
    pairwise_coprime(List.cons(Nat.3, List.cons(Nat.5, List.nil[Nat])))
    pairwise_coprime(system_moduli(crt_example_system))
}

/// Every modulus of the classic example system is positive.
theorem crt_example_every_modulus_positive {
    every_modulus_positive(crt_example_system)
} by {
    Nat.3 != Nat.0
    Nat.5 != Nat.0
    crt_every_modulus_positive_nil
    crt_every_modulus_positive_cons(Pair.new(Nat.5, Nat.3), List.nil[Pair[Nat, Nat]])
    every_modulus_positive(List.cons(Pair.new(Nat.5, Nat.3), List.nil[Pair[Nat, Nat]]))
    crt_every_modulus_positive_cons(Pair.new(Nat.3, Nat.2),
        List.cons(Pair.new(Nat.5, Nat.3), List.nil[Pair[Nat, Nat]]))
    every_modulus_positive(List.cons(Pair.new(Nat.3, Nat.2),
        List.cons(Pair.new(Nat.5, Nat.3), List.nil[Pair[Nat, Nat]])))
    crt_example_system = List.cons(Pair.new(Nat.3, Nat.2),
        List.cons(Pair.new(Nat.5, Nat.3), List.nil[Pair[Nat, Nat]]))
    every_modulus_positive(crt_example_system)
}

/// The combined modulus of the classic example system is 15.
theorem crt_example_system_modulus_is_fifteen {
    system_modulus(crt_example_system) = Nat.15
} by {
    system_modulus_cons(Pair.new(Nat.3, Nat.2),
        List.cons(Pair.new(Nat.5, Nat.3), List.nil[Pair[Nat, Nat]]))
    system_modulus(crt_example_system) =
        Nat.3 * system_modulus(List.cons(Pair.new(Nat.5, Nat.3), List.nil[Pair[Nat, Nat]]))
    system_modulus_cons(Pair.new(Nat.5, Nat.3), List.nil[Pair[Nat, Nat]])
    system_modulus(List.cons(Pair.new(Nat.5, Nat.3), List.nil[Pair[Nat, Nat]])) =
        Nat.5 * system_modulus(List.nil[Pair[Nat, Nat]])
    system_modulus(List.nil[Pair[Nat, Nat]]) = Nat.1
    Nat.5 * Nat.1 = Nat.5
    system_modulus(List.cons(Pair.new(Nat.5, Nat.3), List.nil[Pair[Nat, Nat]])) = Nat.5
    Nat.3 * Nat.5 = Nat.15
    system_modulus(crt_example_system) = Nat.15
}

/// The classic example is solved by `8`, and `8` satisfies every congruence
/// of the example system.
theorem crt_example_eight_satisfies_all {
    satisfies_all(Nat.8, crt_example_system)
} by {
    crt_example_eight_solves
    Nat.8.congr_mod(Nat.2, Nat.3)
    Nat.8.congr_mod(Nat.3, Nat.5)
    satisfies_all_nil(Nat.8)
    satisfies_all(Nat.8, List.nil[Pair[Nat, Nat]])
    satisfies_all_cons_intro(Nat.8, Pair.new(Nat.5, Nat.3), List.nil[Pair[Nat, Nat]])
    satisfies_all(Nat.8, List.cons(Pair.new(Nat.5, Nat.3), List.nil[Pair[Nat, Nat]]))
    satisfies_all_cons_intro(Nat.8, Pair.new(Nat.3, Nat.2),
        List.cons(Pair.new(Nat.5, Nat.3), List.nil[Pair[Nat, Nat]]))
    satisfies_all(Nat.8, List.cons(Pair.new(Nat.3, Nat.2),
        List.cons(Pair.new(Nat.5, Nat.3), List.nil[Pair[Nat, Nat]])))
    crt_example_system = List.cons(Pair.new(Nat.3, Nat.2),
        List.cons(Pair.new(Nat.5, Nat.3), List.nil[Pair[Nat, Nat]]))
    satisfies_all(Nat.8, crt_example_system)
}

/// The canonical CRT solution of the classic example system is `8`: `8`
/// satisfies the system and is normalized below the combined modulus, and
/// normalized solutions are exactly the canonical solution.
theorem crt_example_canonical_solution_is_eight {
    crt_canonical_solution(crt_example_system) = Nat.8
} by {
    crt_example_pairwise_coprime
    crt_example_every_modulus_positive
    crt_example_eight_satisfies_all
    crt_example_system_modulus_is_fifteen
    crt_example_8_lt_15
    system_modulus(crt_example_system) = Nat.15
    Nat.8 < system_modulus(crt_example_system)
    normalized_solution_eq_crt_canonical(crt_example_system, Nat.8)
    Nat.8 = crt_canonical_solution(crt_example_system)
    crt_canonical_solution(crt_example_system) = Nat.8
}

// ---------------------------------------------------------------------------
// The general system: pairwise-coprime moduli, unique solution mod m₁⋯mₖ.
// ---------------------------------------------------------------------------

/// The list CRT, restated: a system of pairwise-coprime positive moduli with
/// residues has a simultaneous solution, and any two solutions are congruent
/// modulo the combined modulus `m₁⋯mₖ`.  This combines `nat_crt_list` with
/// `satisfies_all_unique_mod_system_modulus` from crt_list.ac.
theorem crt_system_solution_unique(system: List[Pair[Nat, Nat]]) {
    pairwise_coprime(system_moduli(system)) and every_modulus_positive(system)
        implies exists(c: Nat) {
            satisfies_all(c, system)
                and forall(c0: Nat) {
                    satisfies_all(c0, system) implies c0.congr_mod(c, system_modulus(system))
                }
        }
} by {
    if pairwise_coprime(system_moduli(system)) and every_modulus_positive(system) {
        nat_crt_list(system)
        let c: Nat satisfy { satisfies_all(c, system) }
        forall(c0: Nat) {
            satisfies_all_unique_mod_system_modulus(system, c0, c)
            satisfies_all(c0, system) implies c0.congr_mod(c, system_modulus(system))
        }
        satisfies_all(c, system)
        exists(result: Nat) {
            satisfies_all(result, system)
                and forall(c0: Nat) {
                    satisfies_all(c0, system) implies c0.congr_mod(result, system_modulus(system))
                }
        }
    }
}

// ---------------------------------------------------------------------------
// Euler's totient via the CRT.
// ---------------------------------------------------------------------------

/// The CRT pairing preserves units: for coprime `m`, `n`, if `c` is a
/// simultaneous solution `c ≡ a (mod m)`, `c ≡ b (mod n)` with `a < m` and
/// `b < n`, then `c` is coprime to `m * n` exactly when `a` is coprime to
/// `m` and `b` is coprime to `n`.  This is the unit-level content of the
/// isomorphism `Z/(mn) ≅ Z/m × Z/n`: counting these pairs through the
/// pairing gives `φ(mn) = φ(m)φ(n)`.
theorem crt_pair_preserves_units(m: Nat, n: Nat, c: Nat, a: Nat, b: Nat) {
    m.coprime(n) and a < m and b < n
        and c.congr_mod(a, m) and c.congr_mod(b, n)
        implies (c.coprime(m * n) = (a.coprime(m) and b.coprime(n)))
} by {
    if m.coprime(n) and a < m and b < n
        and c.congr_mod(a, m) and c.congr_mod(b, n) {
        small_mod(a, m)
        small_mod(b, n)
        a.mod(m) = a
        b.mod(n) = b
        c.congr_mod(a, m)
        c.mod(m) = a.mod(m)
        c.mod(m) = a
        c.congr_mod(b, n)
        c.mod(n) = b.mod(n)
        c.mod(n) = b
        if c.coprime(m * n) {
            coprime_mul_iff(c, m, n)
            c.coprime(m)
            c.coprime(n)
            coprime_mod_imp(c, m)
            c.mod(m).coprime(m)
            a.coprime(m)
            coprime_mod_imp(c, n)
            c.mod(n).coprime(n)
            b.coprime(n)
            a.coprime(m) and b.coprime(n)
        }
        if a.coprime(m) and b.coprime(n) {
            coprime_mod_iff(c, m)
            c.coprime(m) = c.mod(m).coprime(m)
            a.coprime(m)
            c.mod(m) = a
            c.mod(m).coprime(m)
            c.coprime(m)
            coprime_mod_iff(c, n)
            c.coprime(n) = c.mod(n).coprime(n)
            b.coprime(n)
            c.mod(n) = b
            c.mod(n).coprime(n)
            c.coprime(n)
            coprime_mul(c, m, n)
            c.coprime(m * n)
        }
        c.coprime(m * n) = (a.coprime(m) and b.coprime(n))
    }
}

/// Euler's totient via the CRT: for coprime `m`, `n`, `φ(mn) = φ(m)φ(n)`.
/// The classical proof counts the units of `Z/(mn)` through the CRT pairing
/// of `crt_pair_preserves_units`; the counting identity is proved in
/// `totient.ac` as `nat_totient_mul_coprime` and restated here.
theorem totient_mul_coprime_crt(m: Nat, n: Nat) {
    m.coprime(n) implies (m * n).totient = m.totient * n.totient
} by {
    if m.coprime(n) {
        nat_totient_mul_coprime(m, n)
        nat_totient(m * n) = nat_totient(m) * nat_totient(n)
        nat_totient(m * n) = (m * n).totient
        nat_totient(m) = m.totient
        nat_totient(n) = n.totient
        (m * n).totient = m.totient * n.totient
    }
}

// ---------------------------------------------------------------------------
// The structure of Z/(mn): the CRT pairing.
// ---------------------------------------------------------------------------

/// The CRT pairing exists: for coprime positive `m`, `n` and residues
/// `a < m`, `b < n`, there is a normalized simultaneous solution `c < m * n`
/// with `c ≡ a (mod m)` and `c ≡ b (mod n)`.
theorem crt_pairing_exists(m: Nat, n: Nat, a: Nat, b: Nat) {
    m.coprime(n) and Nat.0 < m and Nat.0 < n and a < m and b < n
        implies exists(c: Nat) {
            c < m * n and c.congr_mod(a, m) and c.congr_mod(b, n)
        }
} by {
    if m.coprime(n) and Nat.0 < m and Nat.0 < n and a < m and b < n {
        m != Nat.0
        n != Nat.0
        nat_crt_two_moduli(m, n, a, b)
        let c0: Nat satisfy { c0.congr_mod(a, m) and c0.congr_mod(b, n) }
        let c: Nat = c0.mod(m * n)
        mul_to_zero(m, n)
        m * n != Nat.0
        mod_lt(c0, m * n)
        c < m * n
        mod_congr_mod_self(c0, m * n)
        c.congr_mod(c0, m * n)
        divides_self(m)
        divides_mul(m, n, m)
        m.divides(m * n)
        nat_congr_mod_descend(m, m * n, c, c0)
        c.congr_mod(c0, m)
        congr_mod_trans(c, c0, a, m)
        c.congr_mod(a, m)
        divides_self(n)
        divides_mul(n, m, n)
        n.divides(n * m)
        n * m = m * n
        n.divides(m * n)
        nat_congr_mod_descend(n, m * n, c, c0)
        c.congr_mod(c0, n)
        congr_mod_trans(c, c0, b, n)
        c.congr_mod(b, n)
        c < m * n and c.congr_mod(a, m) and c.congr_mod(b, n)
        exists(result: Nat) {
            result < m * n and result.congr_mod(a, m) and result.congr_mod(b, n)
        }
    }
}

/// The CRT pairing is unique: two normalized simultaneous solutions below
/// `m * n` coincide.
theorem crt_pairing_unique(m: Nat, n: Nat, c1: Nat, c2: Nat, a: Nat, b: Nat) {
    m.coprime(n) and Nat.0 < m and Nat.0 < n and a < m and b < n
        and c1 < m * n and c1.congr_mod(a, m) and c1.congr_mod(b, n)
        and c2 < m * n and c2.congr_mod(a, m) and c2.congr_mod(b, n)
        implies c1 = c2
} by {
    if m.coprime(n) and Nat.0 < m and Nat.0 < n and a < m and b < n
        and c1 < m * n and c1.congr_mod(a, m) and c1.congr_mod(b, n)
        and c2 < m * n and c2.congr_mod(a, m) and c2.congr_mod(b, n) {
        congr_mod_symm(c2, a, m)
        a.congr_mod(c2, m)
        congr_mod_trans(c1, a, c2, m)
        c1.congr_mod(c2, m)
        congr_mod_symm(c2, b, n)
        b.congr_mod(c2, n)
        congr_mod_trans(c1, b, c2, n)
        c1.congr_mod(c2, n)
        nat_congr_combine_coprime(m, n, c1, c2)
        c1.congr_mod(c2, m * n)
        small_mod(c1, m * n)
        c1.mod(m * n) = c1
        small_mod(c2, m * n)
        c2.mod(m * n) = c2
        c1.congr_mod(c2, m * n)
        c1.mod(m * n) = c2.mod(m * n)
        c1 = c1.mod(m * n)
        c1 = c2
    }
}

/// The structure of `Z/(mn)`: for coprime positive `m`, `n`, every residue
/// pair `(a, b)` with `a < m`, `b < n` has exactly one normalized
/// simultaneous solution below `m * n`.  This is the set-level content of
/// the isomorphism `Z/(mn) ≅ Z/m × Z/n`; the ring operations are compatible
/// with the pairing through `congr_mod_add` and `congr_mod_mul`, and the
/// units correspond by `crt_pair_preserves_units`.
theorem crt_pairing_bijection(m: Nat, n: Nat, a: Nat, b: Nat) {
    m.coprime(n) and Nat.0 < m and Nat.0 < n and a < m and b < n
        implies exists(c: Nat) {
            c < m * n and c.congr_mod(a, m) and c.congr_mod(b, n)
                and forall(c0: Nat) {
                    c0 < m * n and c0.congr_mod(a, m) and c0.congr_mod(b, n) implies c0 = c
                }
        }
} by {
    if m.coprime(n) and Nat.0 < m and Nat.0 < n and a < m and b < n {
        crt_pairing_exists(m, n, a, b)
        let c: Nat satisfy {
            c < m * n and c.congr_mod(a, m) and c.congr_mod(b, n)
        }
        forall(c0: Nat) {
            crt_pairing_unique(m, n, c0, c, a, b)
            c0 < m * n and c0.congr_mod(a, m) and c0.congr_mod(b, n) implies c0 = c
        }
        c < m * n and c.congr_mod(a, m) and c.congr_mod(b, n)
        exists(result: Nat) {
            result < m * n and result.congr_mod(a, m) and result.congr_mod(b, n)
                and forall(c0: Nat) {
                    c0 < m * n and c0.congr_mod(a, m) and c0.congr_mod(b, n) implies c0 = result
                }
        }
    }
}
