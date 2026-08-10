/// Farey sequences: the sorted list of reduced fractions in [0, 1] whose
/// denominators are at most n, built by the classical mediant construction.

from nat import Nat, distrib_left, distrib_right, mul_comm, add_comm, add_assoc,
    add_cancels_right, sub_lt, add_sub, lt_add_left, lt_or_lte, lte_add_left
from pair import Pair
from list import List
numerals Nat

/// The mediant of the fractions a/b and c/d, i.e. (a + c)/(b + d).
define mediant(p: Pair[Nat, Nat], q: Pair[Nat, Nat]) -> Pair[Nat, Nat] {
    Pair.new(p.first + q.first, p.second + q.second)
}

/// The determinant b·c − a·d of the two fractions a/b and c/d.  It is
/// positive exactly when a/b < c/d, and it equals one for consecutive Farey
/// terms.
define det(p: Pair[Nat, Nat], q: Pair[Nat, Nat]) -> Nat {
    p.second * q.first - p.first * q.second
}

/// The list obtained from `l` by inserting, between every consecutive pair
/// (a/b, c/d) whose denominators sum to `n`, their mediant (a + c)/(b + d).
define insert_mediants(n: Nat, l: List[Pair[Nat, Nat]]) -> List[Pair[Nat, Nat]] {
    match l {
        List.nil {
            List.nil[Pair[Nat, Nat]]
        }
        List.cons(h, t) {
            match t {
                List.nil {
                    List.cons(h, List.nil[Pair[Nat, Nat]])
                }
                List.cons(h2, t2) {
                    if h.second + h2.second = n {
                        List.cons(h, List.cons(mediant(h, h2), insert_mediants(n, t)))
                    } else {
                        List.cons(h, insert_mediants(n, t))
                    }
                }
            }
        }
    }
}

/// The Farey sequence of order n: every reduced fraction a/b in [0, 1] with
/// 0 ≤ a ≤ b ≤ n and gcd(a, b) = 1, in increasing order of value.  Built by
/// the classical mediant recursion: F(0) = [0/1, 1/1], and F(n) arises from
/// F(n − 1) by inserting the mediant of each consecutive pair whose
/// denominators sum to n.
define farey(n: Nat) -> List[Pair[Nat, Nat]] {
    match n {
        Nat.zero {
            List.cons(Pair.new(Nat.0, Nat.1), List.cons(Pair.new(Nat.1, Nat.1), List.nil[Pair[Nat, Nat]]))
        }
        Nat.suc(m) {
            insert_mediants(n, farey(m))
        }
    }
}

/// True if the fractions x and y occur consecutively, in this order, in l.
define is_consecutive(l: List[Pair[Nat, Nat]], x: Pair[Nat, Nat], y: Pair[Nat, Nat]) -> Bool {
    match l {
        List.nil {
            false
        }
        List.cons(h, t) {
            match t {
                List.nil {
                    false
                }
                List.cons(h2, t2) {
                    if h = x {
                        if h2 = y {
                            true
                        } else {
                            is_consecutive(t, x, y)
                        }
                    } else {
                        is_consecutive(t, x, y)
                    }
                }
            }
        }
    }
}

/// Recurrence: if the denominators of the first two fractions sum to n, their
/// mediant is inserted between them.
theorem insert_mediants_split(n: Nat, h: Pair[Nat, Nat], h2: Pair[Nat, Nat], t: List[Pair[Nat, Nat]]) {
    h.second + h2.second = n implies insert_mediants(n, List.cons(h, List.cons(h2, t))) = List.cons(h, List.cons(mediant(h, h2), insert_mediants(n, List.cons(h2, t))))
}

/// Recurrence: if the denominators of the first two fractions do not sum to n,
/// the list is kept as is and the recursion continues on the tail.
theorem insert_mediants_keep(n: Nat, h: Pair[Nat, Nat], h2: Pair[Nat, Nat], t: List[Pair[Nat, Nat]]) {
    h.second + h2.second != n implies insert_mediants(n, List.cons(h, List.cons(h2, t))) = List.cons(h, insert_mediants(n, List.cons(h2, t)))
}

/// Subtracting the same summand on both sides cancels it.
theorem sub_cancel_add(a: Nat, b: Nat, c: Nat) {
    (a + b) - (a + c) = b - c
} by {
    lt_or_lte(b, c)
    if b < c {
        lt_add_left(a, b, c)
        a + b < a + c
        sub_lt(a + b, a + c)
        (a + b) - (a + c) = Nat.0
        sub_lt(b, c)
        b - c = Nat.0
        (a + b) - (a + c) = b - c
    } else {
        c <= b
        lte_add_left(a, c, b)
        a + c <= a + b
        add_sub(a + c, a + b)
        (a + b) - (a + c) + (a + c) = a + b
        add_sub(c, b)
        b - c + c = b
        add_comm(a, c)
        a + c = c + a
        (b - c) + (a + c) = (b - c) + (c + a)
        add_assoc(b - c, c, a)
        (b - c) + (c + a) = ((b - c) + c) + a
        ((b - c) + c) + a = b + a
        add_comm(b, a)
        b + a = a + b
        (b - c) + (a + c) = a + b
        (a + b) - (a + c) + (a + c) = (b - c) + (a + c)
        add_cancels_right(a + c, (a + b) - (a + c), b - c)
        (a + b) - (a + c) = b - c
    }
}

/// Subtracting the same summand on both sides cancels it, with the summand on
/// the right.
theorem sub_cancel_add_right(a: Nat, b: Nat, c: Nat) {
    (b + a) - (c + a) = b - c
} by {
    add_comm(b, a)
    b + a = a + b
    add_comm(c, a)
    c + a = a + c
    (b + a) - (c + a) = (a + b) - (a + c)
    sub_cancel_add(a, b, c)
    (a + b) - (a + c) = b - c
    (b + a) - (c + a) = b - c
}

/// The determinant of a fraction and the mediant of it with another fraction
/// is the same as the determinant of the two fractions.
theorem det_mediant_left(p: Pair[Nat, Nat], q: Pair[Nat, Nat]) {
    det(p, mediant(p, q)) = det(p, q)
} by {
    mediant(p, q) = Pair.new(p.first + q.first, p.second + q.second)
    det(p, mediant(p, q)) = p.second * mediant(p, q).first - p.first * mediant(p, q).second
    mediant(p, q).first = p.first + q.first
    mediant(p, q).second = p.second + q.second
    p.second * mediant(p, q).first - p.first * mediant(p, q).second = p.second * (p.first + q.first) - p.first * (p.second + q.second)
    distrib_left(p.second, p.first, q.first)
    p.second * (p.first + q.first) = p.second * p.first + p.second * q.first
    distrib_left(p.first, p.second, q.second)
    p.first * (p.second + q.second) = p.first * p.second + p.first * q.second
    mul_comm(p.second, p.first)
    p.second * p.first = p.first * p.second
    p.second * p.first + p.second * q.first = p.first * p.second + p.second * q.first
    sub_cancel_add(p.first * p.second, p.second * q.first, p.first * q.second)
    (p.first * p.second + p.second * q.first) - (p.first * p.second + p.first * q.second) = p.second * q.first - p.first * q.second
    p.second * (p.first + q.first) - p.first * (p.second + q.second) = p.second * q.first - p.first * q.second
    det(p, mediant(p, q)) = p.second * q.first - p.first * q.second
    det(p, q) = p.second * q.first - p.first * q.second
}

/// The determinant of the mediant of two fractions and the second fraction is
/// the same as the determinant of the two fractions.
theorem det_mediant_right(p: Pair[Nat, Nat], q: Pair[Nat, Nat]) {
    det(mediant(p, q), q) = det(p, q)
} by {
    mediant(p, q) = Pair.new(p.first + q.first, p.second + q.second)
    det(mediant(p, q), q) = mediant(p, q).second * q.first - mediant(p, q).first * q.second
    mediant(p, q).first = p.first + q.first
    mediant(p, q).second = p.second + q.second
    mediant(p, q).second * q.first - mediant(p, q).first * q.second = (p.second + q.second) * q.first - (p.first + q.first) * q.second
    distrib_right(p.second, q.second, q.first)
    (p.second + q.second) * q.first = p.second * q.first + q.second * q.first
    distrib_right(p.first, q.first, q.second)
    (p.first + q.first) * q.second = p.first * q.second + q.first * q.second
    mul_comm(q.second, q.first)
    q.second * q.first = q.first * q.second
    p.second * q.first + q.second * q.first = p.second * q.first + q.first * q.second
    (p.second * q.first + q.second * q.first) - (p.first * q.second + q.first * q.second) = (p.second * q.first + q.first * q.second) - (p.first * q.second + q.first * q.second)
    sub_cancel_add_right(q.first * q.second, p.second * q.first, p.first * q.second)
    (p.second * q.first + q.first * q.second) - (p.first * q.second + q.first * q.second) = p.second * q.first - p.first * q.second
    (p.second + q.second) * q.first - (p.first + q.first) * q.second = p.second * q.first - p.first * q.second
    det(mediant(p, q), q) = p.second * q.first - p.first * q.second
    det(p, q) = p.second * q.first - p.first * q.second
}

/// The Farey sequence of order 1 is [0/1, 1/1].
theorem farey_one {
    farey(Nat.1) = List.cons(Pair.new(Nat.0, Nat.1), List.cons(Pair.new(Nat.1, Nat.1), List.nil[Pair[Nat, Nat]]))
} by {
    farey(Nat.1) = insert_mediants(Nat.1, farey(Nat.0))
    farey(Nat.0) = List.cons(Pair.new(Nat.0, Nat.1), List.cons(Pair.new(Nat.1, Nat.1), List.nil[Pair[Nat, Nat]]))
    Pair.new(Nat.0, Nat.1).second = Nat.1
    Pair.new(Nat.1, Nat.1).second = Nat.1
    Nat.1 + Nat.1 != Nat.1
    Pair.new(Nat.0, Nat.1).second + Pair.new(Nat.1, Nat.1).second != Nat.1
    insert_mediants_keep(Nat.1, Pair.new(Nat.0, Nat.1), Pair.new(Nat.1, Nat.1), List.nil[Pair[Nat, Nat]])
    insert_mediants(Nat.1, List.cons(Pair.new(Nat.0, Nat.1), List.cons(Pair.new(Nat.1, Nat.1), List.nil[Pair[Nat, Nat]]))) = List.cons(Pair.new(Nat.0, Nat.1), insert_mediants(Nat.1, List.cons(Pair.new(Nat.1, Nat.1), List.nil[Pair[Nat, Nat]])))
    insert_mediants(Nat.1, List.cons(Pair.new(Nat.1, Nat.1), List.nil[Pair[Nat, Nat]])) = List.cons(Pair.new(Nat.1, Nat.1), List.nil[Pair[Nat, Nat]])
}

/// The Farey sequence of order 2 is [0/1, 1/2, 1/1].
theorem farey_two {
    farey(Nat.2) = List.cons(Pair.new(Nat.0, Nat.1), List.cons(Pair.new(Nat.1, Nat.2), List.cons(Pair.new(Nat.1, Nat.1), List.nil[Pair[Nat, Nat]])))
} by {
    farey(Nat.2) = insert_mediants(Nat.2, farey(Nat.1))
    Pair.new(Nat.0, Nat.1).second + Pair.new(Nat.1, Nat.1).second = Nat.2
    insert_mediants_split(Nat.2, Pair.new(Nat.0, Nat.1), Pair.new(Nat.1, Nat.1), List.nil[Pair[Nat, Nat]])
    insert_mediants(Nat.2, List.cons(Pair.new(Nat.0, Nat.1), List.cons(Pair.new(Nat.1, Nat.1), List.nil[Pair[Nat, Nat]]))) = List.cons(Pair.new(Nat.0, Nat.1), List.cons(mediant(Pair.new(Nat.0, Nat.1), Pair.new(Nat.1, Nat.1)), insert_mediants(Nat.2, List.cons(Pair.new(Nat.1, Nat.1), List.nil[Pair[Nat, Nat]]))))
    mediant(Pair.new(Nat.0, Nat.1), Pair.new(Nat.1, Nat.1)) = Pair.new(Nat.1, Nat.2)
    insert_mediants(Nat.2, List.cons(Pair.new(Nat.1, Nat.1), List.nil[Pair[Nat, Nat]])) = List.cons(Pair.new(Nat.1, Nat.1), List.nil[Pair[Nat, Nat]])
}

/// The consecutive pair 0/1 < 1/2 of the Farey sequence of order 2 has
/// determinant b·c − a·d = 1·1 − 0·2 = 1.
theorem det_zero_one_one_two {
    det(Pair.new(Nat.0, Nat.1), Pair.new(Nat.1, Nat.2)) = Nat.1
} by {
    det(Pair.new(Nat.0, Nat.1), Pair.new(Nat.1, Nat.2)) = Pair.new(Nat.0, Nat.1).second * Pair.new(Nat.1, Nat.2).first - Pair.new(Nat.0, Nat.1).first * Pair.new(Nat.1, Nat.2).second
    Pair.new(Nat.0, Nat.1).second = Nat.1
    Pair.new(Nat.1, Nat.2).first = Nat.1
    Pair.new(Nat.0, Nat.1).first = Nat.0
    Pair.new(Nat.1, Nat.2).second = Nat.2
    det(Pair.new(Nat.0, Nat.1), Pair.new(Nat.1, Nat.2)) = Nat.1 * Nat.1 - Nat.0 * Nat.2
    det(Pair.new(Nat.0, Nat.1), Pair.new(Nat.1, Nat.2)) = Nat.1 - Nat.0
    det(Pair.new(Nat.0, Nat.1), Pair.new(Nat.1, Nat.2)) = Nat.1
}

/// The consecutive pair 1/2 < 1/1 of the Farey sequence of order 2 has
/// determinant 2·1 − 1·1 = 1: b·c − a·d = 1.
theorem det_one_two_one_one {
    det(Pair.new(Nat.1, Nat.2), Pair.new(Nat.1, Nat.1)) = Nat.1
} by {
    det(Pair.new(Nat.1, Nat.2), Pair.new(Nat.1, Nat.1)) = Pair.new(Nat.1, Nat.2).second * Pair.new(Nat.1, Nat.1).first - Pair.new(Nat.1, Nat.2).first * Pair.new(Nat.1, Nat.1).second
    Pair.new(Nat.1, Nat.2).second = Nat.2
    Pair.new(Nat.1, Nat.1).first = Nat.1
    Pair.new(Nat.1, Nat.2).first = Nat.1
    Pair.new(Nat.1, Nat.1).second = Nat.1
    det(Pair.new(Nat.1, Nat.2), Pair.new(Nat.1, Nat.1)) = Nat.2 * Nat.1 - Nat.1 * Nat.1
    det(Pair.new(Nat.1, Nat.2), Pair.new(Nat.1, Nat.1)) = Nat.2 - Nat.1
    det(Pair.new(Nat.1, Nat.2), Pair.new(Nat.1, Nat.1)) = Nat.1
}

/// The pair (x, y) of fractions appearing in the Farey sequence of order n as
/// consecutive terms, characterised recursively along the mediant
/// construction: the pairs of F(n) are those of F(n − 1) that are kept, plus
/// the two pairs (p, m) and (m, q) obtained by inserting the mediant m of a
/// consecutive pair (p, q) of F(n − 1) with b(p) + b(q) = n.  Here the
/// "un-mediants" recover p and q from the split pairs by subtraction.
define unmediant_left(x: Pair[Nat, Nat], y: Pair[Nat, Nat]) -> Pair[Nat, Nat] {
    Pair.new(y.first - x.first, y.second - x.second)
}

/// The other un-mediant: recovers the first fraction of a right split pair.
define unmediant_right(x: Pair[Nat, Nat], y: Pair[Nat, Nat]) -> Pair[Nat, Nat] {
    Pair.new(x.first - y.first, x.second - y.second)
}

/// True if x/y and y are consecutive terms of the Farey sequence of order n,
/// as built by the classical mediant recursion.
define cons_pair_farey(n: Nat, x: Pair[Nat, Nat], y: Pair[Nat, Nat]) -> Bool {
    match n {
        Nat.zero {
            x = Pair.new(Nat.0, Nat.1) and y = Pair.new(Nat.1, Nat.1)
        }
        Nat.suc(m) {
            (cons_pair_farey(m, x, y) and x.second + y.second != Nat.suc(m)) or
            ((cons_pair_farey(m, x, unmediant_left(x, y)) and y = mediant(x, unmediant_left(x, y)) and x.second + unmediant_left(x, y).second = Nat.suc(m)) or
             (cons_pair_farey(m, unmediant_right(x, y), y) and x = mediant(unmediant_right(x, y), y) and unmediant_right(x, y).second + y.second = Nat.suc(m)))
        }
    }
}

// The fundamental property: consecutive terms a/b < c/d of the Farey sequence
// of order n satisfy b·c − a·d = 1.  The proof is the classical induction on
// n over the mediant construction: the mediant of a consecutive pair (p, q)
// with b(p) + b(q) = n splits the pair into (p, mediant(p, q)) and
// (mediant(p, q), q), and det(p, mediant(p, q)) = det(p, q) (see
// det_mediant_left and det_mediant_right), so the invariant det = 1 is
// preserved.  The case analysis this requires (splitting the recursive
// predicate into its three cases and eliminating the disjunction) does not
// yet verify within the prover's search budget, so the theorem is left
// stated below.
//
// theorem farey_consecutive_det_one(n: Nat) {
//     forall(x: Pair[Nat, Nat], y: Pair[Nat, Nat]) {
//         cons_pair_farey(n, x, y) implies det(x, y) = Nat.1
//     }
// }

// Corollary of the fundamental property: the mediant of consecutive terms
// a/b, c/d of F(n) has denominator b + d > n, so the mediant (which is
// reduced when b·c − a·d = 1) cannot itself occur in F(n).  The proof needs
// the same case analysis as the fundamental property.
//
// theorem farey_consecutive_denom_gt(n: Nat) {
//     forall(x: Pair[Nat, Nat], y: Pair[Nat, Nat]) {
//         cons_pair_farey(n, x, y) implies n < x.second + y.second
//     }
// }

// The counting formula: |F(n)| = 1 + sum_{k = 1}^{n} phi(k), where phi is
// Euler's totient.  Along the mediant construction F(n) adds exactly the
// reduced fractions with denominator n, of which there are phi(n), so
// |F(n)| = |F(n − 1)| + phi(n) and the formula follows by unrolling; the
// bijection between inserted mediants and reduced fractions of denominator n
// (every reduced a/n with gcd(a, n) = 1 is the mediant of the two fractions
// of F(n − 1) adjacent to it) is the deep counting step and is not yet
// formalised.  The statement uses the library's totient, which is
// `Nat.totient` from number_theory.totient.
//
// theorem farey_length_formula(n: Nat) {
//     farey(n).length = Nat.1 + sum(map(Nat.range(n.suc), function(k: Nat) { k.totient }))
// }

// Small-order instances of the fundamental property (by direct computation)
// are also left to the same case analysis.
// theorem farey_two_consecutive_det_one {
//     forall(x: Pair[Nat, Nat], y: Pair[Nat, Nat]) {
//         cons_pair_farey(Nat.2, x, y) implies det(x, y) = Nat.1
//     }
// }
