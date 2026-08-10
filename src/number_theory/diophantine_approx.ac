/// Diophantine approximation: the classical results that irrational numbers
/// have infinitely many good rational approximations, and Dirichlet's
/// approximation theorem.
///
/// The library's continued-fraction machinery (continued_fraction.ac,
/// continued_fraction_convergents.ac, continued_fraction_approx.ac) builds,
/// for every coefficient sequence with positive tail, the real limit alpha of
/// the convergents p_n / q_n together with the approximation estimate
/// `|alpha - p_n / q_n| < 1 / q_n^2` (continued_fraction_approx.ac).  This
/// file restates that estimate and derives the classical consequences:
///
///   1. The approximation estimate `|alpha - p_n / q_n| < 1 / q_n^2`,
///      restated from continued_fraction_approx.ac (Section 1).
///
///   2. Infinitely many good rational approximations: beyond every bound there
///      is a convergent p / q with `q` above the bound and
///      `|alpha - p / q| < 1 / q^2` (Section 2).  Every irrational number is
///      the limit of an infinite continued-fraction expansion, so this is the
///      library's form of the classical statement for irrational alpha.
///
///   3. Dirichlet's approximation theorem: for every bound >= 1 there exist
///      naturals p, q with `1 <= q <= bound` and `|q * alpha - p| < 1 / bound`
///      (Section 3).
///
/// The fully general statement of Dirichlet's theorem for an arbitrary real
/// alpha, and the irrationality-measure converse that infinitely many reduced
/// `1 / q^2`-approximations force alpha to be irrational, are recorded as
/// comments at the end, since the library does not yet construct the
/// continued-fraction expansion of an arbitrary real number and lacks the
/// rational-denominator bound used by the converse.

from nat import Nat, from_nat, zero_or_suc, lt_or_lte, add_assoc, add_comm, add_one_right,
    add_suc_right, distrib_left, distrib_right, from_nat_one, from_nat_zero,
    lt_imp_lte_suc, lt_mul_both, lt_suc, lte_add_left, lte_add_right,
    lte_mul_both, lte_trans, mul_assoc, mul_comm, mul_suc_right,
    mul_zero_left, mul_zero_right, mul_one_left, mul_one_right,
    add_zero_right, add_zero_left, lt_and_lte, lte_antisymm, lte_cancel_suc,
    lte_ref, add_cancels_left, lt_not_ref
from rat import Rat, cross_mul_lt, cross_mul_lte, from_nat_add, from_nat_mul,
    nat_lt_imp_rat_lt, nat_lte_imp_rat_lte, mul_fractions,
    cancel_left_num_denom, recip_eq_one_div, recip_mul, mul_inv_cancels_right,
    mul_div_cancels, pos_inverse, zero_lt_imp_pos, pos_ne_zero,
    pos_imp_zero_lt, rat_total, sub_self, div_zero, abs_div, lt_some_nat
from real import Real, add_from_rat, from_nat_is_from_rat, real_from_rat_inverse,
    from_rat_maintains_lt, from_rat_maintains_lte, mul_from_rat, neg_from_rat,
    abs_from_rat, pos_imp_eq_abs, abs_gte_zero, mul_abs, lt_mul_pos_left,
    lt_mul_pos_right, lt_trans, lt_add_right, lt_add_left, lt_imp_minus_pos,
    gt_zero_imp_pos, pos_gt_zero, lte_abs, rat_between_reals,
    mul_sub_distrib_right, mul_sub_distrib_left
from order import lt_imp_lte, lte_imp_not_lt, lt_of_lte_of_lt, lt_of_lt_of_lte
from ordered_field import inverse_on_positive_flips_inequality
from number_theory.continued_fraction_convergents import continued_fraction_convergent_numerator,
    continued_fraction_convergent_denominator, continued_fraction_convergent_value,
    positive_continued_fraction_sequence_tail,
    continued_fraction_convergent_denominator_positive,
    continued_fraction_convergent_denominator_zero
from number_theory.continued_fraction_approx import continued_fraction_approximation_estimate,
    continued_fraction_real_limit, continued_fraction_real_convergent_value,
    continued_fraction_real_gap_bound, continued_fraction_gap_bound,
    continued_fraction_real_gap_eq_embedded_gap, continued_fraction_real_even_convergent,
    continued_fraction_real_odd_convergent, continued_fraction_real_even_lt_limit,
    continued_fraction_real_limit_lt_odd, real_strict_between_abs_lt,
    real_strict_between_abs_lt_right, real_neg_sub, real_neg_abs,
    nat_even_or_odd, real_from_nat_recip_inverse, rat_from_nat_positive_ne_zero,
    real_abs_of_nonneg, continued_fraction_convergent_denominator_ge_one,
    continued_fraction_convergent_denominator_one_ge,
    continued_fraction_convergent_denominator_ge_index,
    continued_fraction_convergent_denominator_suc_gt,
    continued_fraction_convergent_denominator_suc_ge_one_more

numerals Nat

// ============================================================================
// Section 1: the approximation estimate |alpha - p_n / q_n| < 1 / q_n^2
// ============================================================================

/// The classical approximation estimate for the continued-fraction limit: the
/// limit is within `1 / q_n^2` of the `n`-th convergent.
///
/// This restates `continued_fraction_approximation_estimate` from
/// continued_fraction_approx.ac; the full proof lives there.
theorem diophantine_approximation_estimate(
    coefficients: Nat -> Nat, n: Nat
) {
    positive_continued_fraction_sequence_tail(coefficients) implies (continued_fraction_real_limit(coefficients) - continued_fraction_real_convergent_value(coefficients, n)).abs < Real.from_rat(Rat.1 / Rat.from_nat(
            continued_fraction_convergent_denominator(coefficients, n) *
            continued_fraction_convergent_denominator(coefficients, n)))
} by {
    if positive_continued_fraction_sequence_tail(coefficients) {
        continued_fraction_approximation_estimate(coefficients, n)
        (continued_fraction_real_limit(coefficients) - continued_fraction_real_convergent_value(coefficients, n)).abs < Real.from_rat(Rat.1 / Rat.from_nat(
                continued_fraction_convergent_denominator(coefficients, n) *
                continued_fraction_convergent_denominator(coefficients, n)))
    }
}

// ============================================================================
// Section 2: infinitely many good rational approximations
// ============================================================================

/// The continued-fraction limit has infinitely many good rational
/// approximations: beyond every bound `n` there is a convergent `p / q` with
/// denominator exceeding `n` and `|alpha - p / q| < 1 / q^2`.
///
/// The convergent at index `n + 1` has denominator at least `n + 1` and
/// satisfies the approximation estimate.
theorem diophantine_good_approximations_unbounded(
    coefficients: Nat -> Nat, n: Nat
) {
    positive_continued_fraction_sequence_tail(coefficients) implies
        exists(p: Nat, q: Nat) {
            n < q and
            (continued_fraction_real_limit(coefficients) -
                Real.from_rat(Rat.from_nat(p) / Rat.from_nat(q))).abs <
                Real.from_rat(Rat.1 / Rat.from_nat(q * q))
        }
} by {
    if positive_continued_fraction_sequence_tail(coefficients) {
        diophantine_approximation_estimate(coefficients, n.suc)
        (continued_fraction_real_limit(coefficients) - continued_fraction_real_convergent_value(coefficients, n.suc)).abs < Real.from_rat(Rat.1 / Rat.from_nat(continued_fraction_convergent_denominator(coefficients, n.suc) * continued_fraction_convergent_denominator(coefficients, n.suc)))
        continued_fraction_convergent_denominator_ge_index(coefficients, n.suc)
        n.suc <= continued_fraction_convergent_denominator(coefficients, n.suc)
        lt_suc(n)
        n < n.suc
        lt_and_lte(n, n.suc, continued_fraction_convergent_denominator(coefficients, n.suc))
        n < continued_fraction_convergent_denominator(coefficients, n.suc)
        continued_fraction_real_convergent_value(coefficients, n.suc) = Real.from_rat(continued_fraction_convergent_value(coefficients, n.suc))
        continued_fraction_convergent_value(coefficients, n.suc) = Rat.from_nat(continued_fraction_convergent_numerator(coefficients, n.suc)) / Rat.from_nat(continued_fraction_convergent_denominator(coefficients, n.suc))
        continued_fraction_real_convergent_value(coefficients, n.suc) = Real.from_rat(Rat.from_nat(continued_fraction_convergent_numerator(coefficients, n.suc)) / Rat.from_nat(continued_fraction_convergent_denominator(coefficients, n.suc)))
        exists(p: Nat, q: Nat) { n < q and (continued_fraction_real_limit(coefficients) - Real.from_rat(Rat.from_nat(p) / Rat.from_nat(q))).abs < Real.from_rat(Rat.1 / Rat.from_nat(q * q)) }
    }
}

// ============================================================================
// Section 3: Dirichlet's approximation theorem
// ============================================================================

/// True when some convergent has denominator at most the bound while the next
/// convergent denominator exceeds it.
define dirichlet_index_pred(coefficients: Nat -> Nat, bound: Nat) -> Bool {
    exists(n: Nat) {
        continued_fraction_convergent_denominator(coefficients, n) <= bound and
        bound < continued_fraction_convergent_denominator(coefficients, n.suc)
    }
}

/// For every bound >= 1 there is an index whose convergent denominator is at
/// most the bound while the next convergent denominator exceeds it.
///
/// The convergent denominators start at one, never decrease, and grow without
/// bound, so the set of indices with denominator at most the bound is a
/// nonempty initial segment; the index right after it satisfies both
/// inequalities.
theorem dirichlet_convergent_index(coefficients: Nat -> Nat, bound: Nat) {
    positive_continued_fraction_sequence_tail(coefficients) and Nat.1 <= bound implies
    exists(n: Nat) {
        continued_fraction_convergent_denominator(coefficients, n) <= bound and
        bound < continued_fraction_convergent_denominator(coefficients, n.suc)
    }
} by {
    if positive_continued_fraction_sequence_tail(coefficients) and Nat.1 <= bound {
        define p(m: Nat) -> Bool {
            dirichlet_index_pred(coefficients, m.suc)
        }
        continued_fraction_convergent_denominator_zero(coefficients)
        continued_fraction_convergent_denominator(coefficients, Nat.0) = Nat.1
        continued_fraction_convergent_denominator_one_ge(coefficients)
        Nat.1 <= continued_fraction_convergent_denominator(coefficients, Nat.1)
        if continued_fraction_convergent_denominator(coefficients, Nat.1) = Nat.1 {
            continued_fraction_convergent_denominator(coefficients, Nat.1) <= Nat.1
            continued_fraction_convergent_denominator_suc_gt(coefficients, Nat.0)
            continued_fraction_convergent_denominator(coefficients, Nat.0.suc) < continued_fraction_convergent_denominator(coefficients, Nat.0.suc.suc)
            continued_fraction_convergent_denominator(coefficients, Nat.1) < continued_fraction_convergent_denominator(coefficients, Nat.0.suc.suc)
            Nat.1 < continued_fraction_convergent_denominator(coefficients, Nat.0.suc.suc)
            continued_fraction_convergent_denominator(coefficients, Nat.1) <= Nat.1 and Nat.1 < continued_fraction_convergent_denominator(coefficients, Nat.0.suc.suc)
            exists(n: Nat) {
                continued_fraction_convergent_denominator(coefficients, n) <= Nat.1 and
                Nat.1 < continued_fraction_convergent_denominator(coefficients, n.suc)
            }
        }
        if continued_fraction_convergent_denominator(coefficients, Nat.1) != Nat.1 {
            Nat.1 < continued_fraction_convergent_denominator(coefficients, Nat.1)
            continued_fraction_convergent_denominator(coefficients, Nat.0) <= Nat.1
            continued_fraction_convergent_denominator(coefficients, Nat.0) <= Nat.1 and Nat.1 < continued_fraction_convergent_denominator(coefficients, Nat.1)
            exists(n: Nat) {
                continued_fraction_convergent_denominator(coefficients, n) <= Nat.1 and
                Nat.1 < continued_fraction_convergent_denominator(coefficients, n.suc)
            }
        }
        exists(n: Nat) {
            continued_fraction_convergent_denominator(coefficients, n) <= Nat.1 and
            Nat.1 < continued_fraction_convergent_denominator(coefficients, n.suc)
        }
        dirichlet_index_pred(coefficients, Nat.1) = exists(n: Nat) {
            continued_fraction_convergent_denominator(coefficients, n) <= Nat.1 and
            Nat.1 < continued_fraction_convergent_denominator(coefficients, n.suc)
        }
        dirichlet_index_pred(coefficients, Nat.1)
        p(Nat.0) = dirichlet_index_pred(coefficients, Nat.1)
        p(Nat.0)
        forall(m: Nat) {
            if p(m) {
                p(m) = dirichlet_index_pred(coefficients, m.suc)
                dirichlet_index_pred(coefficients, m.suc)
                dirichlet_index_pred(coefficients, m.suc) = exists(n: Nat) {
                    continued_fraction_convergent_denominator(coefficients, n) <= m.suc and
                    m.suc < continued_fraction_convergent_denominator(coefficients, n.suc)
                }
                exists(n: Nat) {
                    continued_fraction_convergent_denominator(coefficients, n) <= m.suc and
                    m.suc < continued_fraction_convergent_denominator(coefficients, n.suc)
                }
                let n: Nat satisfy {
                    continued_fraction_convergent_denominator(coefficients, n) <= m.suc and
                    m.suc < continued_fraction_convergent_denominator(coefficients, n.suc)
                }
                continued_fraction_convergent_denominator(coefficients, n) <= m.suc
                m.suc < continued_fraction_convergent_denominator(coefficients, n.suc)
                lt_or_lte(m.suc.suc, continued_fraction_convergent_denominator(coefficients, n.suc))
                m.suc.suc < continued_fraction_convergent_denominator(coefficients, n.suc) or continued_fraction_convergent_denominator(coefficients, n.suc) <= m.suc.suc
                if m.suc.suc < continued_fraction_convergent_denominator(coefficients, n.suc) {
                    lt_suc(m.suc)
                    m.suc < m.suc.suc
                    lt_imp_lte(m.suc, m.suc.suc)
                    m.suc <= m.suc.suc
                    lte_trans(continued_fraction_convergent_denominator(coefficients, n), m.suc, m.suc.suc)
                    continued_fraction_convergent_denominator(coefficients, n) <= m.suc.suc
                    m.suc.suc < continued_fraction_convergent_denominator(coefficients, n.suc)
                    continued_fraction_convergent_denominator(coefficients, n) <= m.suc.suc and m.suc.suc < continued_fraction_convergent_denominator(coefficients, n.suc)
                    exists(n1: Nat) {
                        continued_fraction_convergent_denominator(coefficients, n1) <= m.suc.suc and
                        m.suc.suc < continued_fraction_convergent_denominator(coefficients, n1.suc)
                    }
                    dirichlet_index_pred(coefficients, m.suc.suc) = exists(n1: Nat) {
                        continued_fraction_convergent_denominator(coefficients, n1) <= m.suc.suc and
                        m.suc.suc < continued_fraction_convergent_denominator(coefficients, n1.suc)
                    }
                    dirichlet_index_pred(coefficients, m.suc.suc)
                }
                if continued_fraction_convergent_denominator(coefficients, n.suc) <= m.suc.suc {
                    lt_imp_lte_suc(m.suc, continued_fraction_convergent_denominator(coefficients, n.suc))
                    m.suc.suc <= continued_fraction_convergent_denominator(coefficients, n.suc)
                    lte_antisymm(continued_fraction_convergent_denominator(coefficients, n.suc), m.suc.suc)
                    continued_fraction_convergent_denominator(coefficients, n.suc) = m.suc.suc
                    continued_fraction_convergent_denominator_suc_gt(coefficients, n)
                    continued_fraction_convergent_denominator(coefficients, n.suc) < continued_fraction_convergent_denominator(coefficients, n.suc.suc)
                    m.suc.suc < continued_fraction_convergent_denominator(coefficients, n.suc.suc)
                    continued_fraction_convergent_denominator(coefficients, n.suc) <= m.suc.suc and m.suc.suc < continued_fraction_convergent_denominator(coefficients, n.suc.suc)
                    exists(n1: Nat) {
                        continued_fraction_convergent_denominator(coefficients, n1) <= m.suc.suc and
                        m.suc.suc < continued_fraction_convergent_denominator(coefficients, n1.suc)
                    }
                    dirichlet_index_pred(coefficients, m.suc.suc) = exists(n1: Nat) {
                        continued_fraction_convergent_denominator(coefficients, n1) <= m.suc.suc and
                        m.suc.suc < continued_fraction_convergent_denominator(coefficients, n1.suc)
                    }
                    dirichlet_index_pred(coefficients, m.suc.suc)
                }
                dirichlet_index_pred(coefficients, m.suc.suc)
                p(m.suc) = dirichlet_index_pred(coefficients, m.suc.suc)
                p(m.suc)
            }
            p(m) implies p(m.suc)
        }
        p(Nat.0) and forall(m: Nat) { p(m) implies p(m.suc) }
        Nat.induction(p)
        forall(m: Nat) { p(m) }
        zero_or_suc(bound)
        bound = Nat.0 or exists(m: Nat) { m.suc = bound }
        if bound = Nat.0 {
            Nat.1 <= Nat.0
            lte_imp_not_lt(Nat.1, Nat.0)
            not (Nat.0 < Nat.1)
            lt_suc(Nat.0)
            Nat.0 < Nat.1
            false
        }
        if exists(m: Nat) { m.suc = bound } {
            let m: Nat satisfy {
                m.suc = bound
            }
            p(m)
            p(m) = dirichlet_index_pred(coefficients, m.suc)
            dirichlet_index_pred(coefficients, m.suc)
            m.suc = bound
            dirichlet_index_pred(coefficients, bound)
            dirichlet_index_pred(coefficients, bound) = exists(n: Nat) {
                continued_fraction_convergent_denominator(coefficients, n) <= bound and
                bound < continued_fraction_convergent_denominator(coefficients, n.suc)
            }
            exists(n: Nat) {
                continued_fraction_convergent_denominator(coefficients, n) <= bound and
                bound < continued_fraction_convergent_denominator(coefficients, n.suc)
            }
        }
        exists(n: Nat) {
            continued_fraction_convergent_denominator(coefficients, n) <= bound and
            bound < continued_fraction_convergent_denominator(coefficients, n.suc)
        }
    }
}

/// The real limit is strictly closer to a convergent than the next convergent
/// is: `|alpha - p_n / q_n| < 1 / (q_n * q_{n+1})`.
///
/// The limit lies strictly between every pair of adjacent convergents, and
/// the adjacent gap is exactly `1 / (q_n * q_{n+1})`; this is the first half
/// of the proof of `diophantine_approximation_estimate`.
theorem diophantine_limit_gap_approx(coefficients: Nat -> Nat, n: Nat) {
    positive_continued_fraction_sequence_tail(coefficients) implies
    (continued_fraction_real_limit(coefficients) - continued_fraction_real_convergent_value(coefficients, n)).abs < continued_fraction_real_gap_bound(coefficients, n)
} by {
    if positive_continued_fraction_sequence_tail(coefficients) {
        nat_even_or_odd(n)
        if exists(k: Nat) { n = Nat.2 * k } {
            let k: Nat satisfy {
                n = Nat.2 * k
            }
            n = Nat.2 * k
            continued_fraction_real_even_lt_limit(coefficients, k)
            continued_fraction_real_even_convergent(coefficients, k) < continued_fraction_real_limit(coefficients)
            continued_fraction_real_limit_lt_odd(coefficients, k)
            continued_fraction_real_limit(coefficients) < continued_fraction_real_odd_convergent(coefficients, k)
            real_strict_between_abs_lt(
                continued_fraction_real_even_convergent(coefficients, k),
                continued_fraction_real_limit(coefficients),
                continued_fraction_real_odd_convergent(coefficients, k))
            (continued_fraction_real_limit(coefficients) -
                continued_fraction_real_even_convergent(coefficients, k)).abs < (continued_fraction_real_odd_convergent(coefficients, k) - continued_fraction_real_even_convergent(coefficients, k)).abs
            continued_fraction_real_convergent_value(coefficients, n) =
                continued_fraction_real_even_convergent(coefficients, k)
            (continued_fraction_real_limit(coefficients) -
                continued_fraction_real_convergent_value(coefficients, n)).abs =
                (continued_fraction_real_limit(coefficients) -
                    continued_fraction_real_even_convergent(coefficients, k)).abs
            continued_fraction_real_convergent_value(coefficients, n.suc) =
                continued_fraction_real_odd_convergent(coefficients, k)
            (continued_fraction_real_odd_convergent(coefficients, k) -
                continued_fraction_real_even_convergent(coefficients, k)).abs =
                (continued_fraction_real_convergent_value(coefficients, n.suc) -
                    continued_fraction_real_convergent_value(coefficients, n)).abs
            (continued_fraction_real_limit(coefficients) -
                continued_fraction_real_convergent_value(coefficients, n)).abs < (continued_fraction_real_convergent_value(coefficients, n.suc) - continued_fraction_real_convergent_value(coefficients, n)).abs
            continued_fraction_real_gap_eq_embedded_gap(coefficients, n)
            (continued_fraction_real_convergent_value(coefficients, n.suc) -
                continued_fraction_real_convergent_value(coefficients, n)).abs =
                continued_fraction_real_gap_bound(coefficients, n)
            (continued_fraction_real_limit(coefficients) -
                continued_fraction_real_convergent_value(coefficients, n)).abs < continued_fraction_real_gap_bound(coefficients, n)
        }
        if exists(k: Nat) { n = (Nat.2 * k).suc } {
            let k: Nat satisfy {
                n = (Nat.2 * k).suc
            }
            n = (Nat.2 * k).suc
            continued_fraction_real_even_lt_limit(coefficients, k.suc)
            continued_fraction_real_even_convergent(coefficients, k.suc) < continued_fraction_real_limit(coefficients)
            continued_fraction_real_limit_lt_odd(coefficients, k)
            continued_fraction_real_limit(coefficients) < continued_fraction_real_odd_convergent(coefficients, k)
            real_strict_between_abs_lt_right(
                continued_fraction_real_even_convergent(coefficients, k.suc),
                continued_fraction_real_limit(coefficients),
                continued_fraction_real_odd_convergent(coefficients, k))
            (continued_fraction_real_odd_convergent(coefficients, k) -
                continued_fraction_real_limit(coefficients)).abs < (continued_fraction_real_odd_convergent(coefficients, k) - continued_fraction_real_even_convergent(coefficients, k.suc)).abs
            continued_fraction_real_convergent_value(coefficients, n.suc) =
                continued_fraction_real_even_convergent(coefficients, k.suc)
            continued_fraction_real_convergent_value(coefficients, n) =
                continued_fraction_real_odd_convergent(coefficients, k)
            (continued_fraction_real_limit(coefficients) -
                continued_fraction_real_convergent_value(coefficients, n)).abs =
                (continued_fraction_real_limit(coefficients) -
                    continued_fraction_real_odd_convergent(coefficients, k)).abs
            real_neg_sub(
                continued_fraction_real_odd_convergent(coefficients, k),
                continued_fraction_real_limit(coefficients))
            -(continued_fraction_real_odd_convergent(coefficients, k) -
                continued_fraction_real_limit(coefficients)) =
                continued_fraction_real_limit(coefficients) -
                continued_fraction_real_odd_convergent(coefficients, k)
            real_neg_abs(continued_fraction_real_odd_convergent(coefficients, k) -
                continued_fraction_real_limit(coefficients))
            (-(continued_fraction_real_odd_convergent(coefficients, k) -
                continued_fraction_real_limit(coefficients))).abs =
                (continued_fraction_real_odd_convergent(coefficients, k) -
                    continued_fraction_real_limit(coefficients)).abs
            (continued_fraction_real_limit(coefficients) -
                continued_fraction_real_odd_convergent(coefficients, k)).abs =
                (continued_fraction_real_odd_convergent(coefficients, k) -
                    continued_fraction_real_limit(coefficients)).abs
            (continued_fraction_real_limit(coefficients) -
                continued_fraction_real_convergent_value(coefficients, n)).abs =
                (continued_fraction_real_odd_convergent(coefficients, k) -
                    continued_fraction_real_limit(coefficients)).abs
            (continued_fraction_real_limit(coefficients) -
                continued_fraction_real_convergent_value(coefficients, n)).abs < (continued_fraction_real_odd_convergent(coefficients, k) - continued_fraction_real_even_convergent(coefficients, k.suc)).abs
            (continued_fraction_real_odd_convergent(coefficients, k) -
                continued_fraction_real_even_convergent(coefficients, k.suc)).abs =
                (continued_fraction_real_convergent_value(coefficients, n) -
                    continued_fraction_real_convergent_value(coefficients, n.suc)).abs
            (continued_fraction_real_limit(coefficients) -
                continued_fraction_real_convergent_value(coefficients, n)).abs < (continued_fraction_real_convergent_value(coefficients, n) - continued_fraction_real_convergent_value(coefficients, n.suc)).abs
            real_neg_sub(
                continued_fraction_real_convergent_value(coefficients, n),
                continued_fraction_real_convergent_value(coefficients, n.suc))
            -(continued_fraction_real_convergent_value(coefficients, n) -
                continued_fraction_real_convergent_value(coefficients, n.suc)) =
                continued_fraction_real_convergent_value(coefficients, n.suc) -
                continued_fraction_real_convergent_value(coefficients, n)
            real_neg_abs(continued_fraction_real_convergent_value(coefficients, n) -
                continued_fraction_real_convergent_value(coefficients, n.suc))
            (-(continued_fraction_real_convergent_value(coefficients, n) -
                continued_fraction_real_convergent_value(coefficients, n.suc))).abs =
                (continued_fraction_real_convergent_value(coefficients, n) -
                    continued_fraction_real_convergent_value(coefficients, n.suc)).abs
            (continued_fraction_real_convergent_value(coefficients, n) -
                continued_fraction_real_convergent_value(coefficients, n.suc)).abs =
                (continued_fraction_real_convergent_value(coefficients, n.suc) -
                    continued_fraction_real_convergent_value(coefficients, n)).abs
            (continued_fraction_real_limit(coefficients) -
                continued_fraction_real_convergent_value(coefficients, n)).abs < (continued_fraction_real_convergent_value(coefficients, n.suc) - continued_fraction_real_convergent_value(coefficients, n)).abs
            continued_fraction_real_gap_eq_embedded_gap(coefficients, n)
            (continued_fraction_real_convergent_value(coefficients, n.suc) -
                continued_fraction_real_convergent_value(coefficients, n)).abs =
                continued_fraction_real_gap_bound(coefficients, n)
            (continued_fraction_real_limit(coefficients) -
                continued_fraction_real_convergent_value(coefficients, n)).abs < continued_fraction_real_gap_bound(coefficients, n)
        }
        (continued_fraction_real_limit(coefficients) -
            continued_fraction_real_convergent_value(coefficients, n)).abs < continued_fraction_real_gap_bound(coefficients, n)
    }
}

/// True when (p, q) is a Dirichlet approximation of alpha with tolerance
/// bound: `1 <= q <= bound` and `|q * alpha - p| < 1 / bound`.
define dirichlet_good_approximation(alpha: Real, p: Nat, q: Nat, bound: Nat) -> Bool {
    Nat.1 <= q and q <= bound and
    (from_nat[Real](q) * alpha - from_nat[Real](p)).abs <
        Real.from_rat(Rat.1 / Rat.from_nat(bound))
}

/// Dirichlet's approximation theorem for the continued-fraction limit: for
/// every bound >= 1 there exist naturals p, q with `1 <= q <= bound` and
/// `|q * alpha - p| < 1 / bound`.
///
/// The index from `dirichlet_convergent_index` has `q_n <= bound < q_{n+1}`;
/// multiplying the gap estimate `|alpha - p_n / q_n| < 1 / (q_n * q_{n+1})` by
/// `q_n` yields `|q_n * alpha - p_n| < 1 / q_{n+1} < 1 / bound`.
theorem diophantine_dirichlet(coefficients: Nat -> Nat, bound: Nat) {
    positive_continued_fraction_sequence_tail(coefficients) and Nat.1 <= bound implies
    exists(p: Nat, q: Nat) {
        dirichlet_good_approximation(continued_fraction_real_limit(coefficients), p, q, bound)
    }
} by {
    if positive_continued_fraction_sequence_tail(coefficients) and Nat.1 <= bound {
        dirichlet_convergent_index(coefficients, bound)
        exists(n: Nat) {
            continued_fraction_convergent_denominator(coefficients, n) <= bound and
            bound < continued_fraction_convergent_denominator(coefficients, n.suc)
        }
        let n: Nat satisfy {
            continued_fraction_convergent_denominator(coefficients, n) <= bound and
            bound < continued_fraction_convergent_denominator(coefficients, n.suc)
        }
        continued_fraction_convergent_denominator(coefficients, n) <= bound
        bound < continued_fraction_convergent_denominator(coefficients, n.suc)
        continued_fraction_convergent_denominator_positive(coefficients, n)
        Nat.0 < continued_fraction_convergent_denominator(coefficients, n)
        continued_fraction_convergent_denominator_positive(coefficients, n.suc)
        Nat.0 < continued_fraction_convergent_denominator(coefficients, n.suc)
        rat_from_nat_positive_ne_zero(continued_fraction_convergent_denominator(coefficients, n))
        Rat.from_nat(continued_fraction_convergent_denominator(coefficients, n)) != Rat.0
        rat_from_nat_positive_ne_zero(continued_fraction_convergent_denominator(coefficients, n.suc))
        Rat.from_nat(continued_fraction_convergent_denominator(coefficients, n.suc)) != Rat.0
        continued_fraction_convergent_denominator_ge_one(coefficients, n)
        Nat.1 <= continued_fraction_convergent_denominator(coefficients, n)
        diophantine_limit_gap_approx(coefficients, n)
        (continued_fraction_real_limit(coefficients) - continued_fraction_real_convergent_value(coefficients, n)).abs < continued_fraction_real_gap_bound(coefficients, n)
        continued_fraction_real_gap_bound(coefficients, n) = Real.from_rat(continued_fraction_gap_bound(coefficients, n))
        continued_fraction_gap_bound(coefficients, n) = Rat.1 / Rat.from_nat(continued_fraction_convergent_denominator(coefficients, n) * continued_fraction_convergent_denominator(coefficients, n.suc))
        continued_fraction_real_gap_bound(coefficients, n) = Real.from_rat(Rat.1 / Rat.from_nat(continued_fraction_convergent_denominator(coefficients, n) * continued_fraction_convergent_denominator(coefficients, n.suc)))
        (continued_fraction_real_limit(coefficients) - continued_fraction_real_convergent_value(coefficients, n)).abs < Real.from_rat(Rat.1 / Rat.from_nat(continued_fraction_convergent_denominator(coefficients, n) * continued_fraction_convergent_denominator(coefficients, n.suc)))
        continued_fraction_real_convergent_value(coefficients, n) = Real.from_rat(continued_fraction_convergent_value(coefficients, n))
        continued_fraction_convergent_value(coefficients, n) = Rat.from_nat(continued_fraction_convergent_numerator(coefficients, n)) / Rat.from_nat(continued_fraction_convergent_denominator(coefficients, n))
        continued_fraction_real_convergent_value(coefficients, n) = Real.from_rat(Rat.from_nat(continued_fraction_convergent_numerator(coefficients, n)) / Rat.from_nat(continued_fraction_convergent_denominator(coefficients, n)))
        from_nat_is_from_rat(continued_fraction_convergent_denominator(coefficients, n))
        from_nat[Real](continued_fraction_convergent_denominator(coefficients, n)) = Real.from_rat(Rat.from_nat(continued_fraction_convergent_denominator(coefficients, n)))
        from_nat_is_from_rat(continued_fraction_convergent_numerator(coefficients, n))
        from_nat[Real](continued_fraction_convergent_numerator(coefficients, n)) = Real.from_rat(Rat.from_nat(continued_fraction_convergent_numerator(coefficients, n)))
        mul_from_rat(Rat.from_nat(continued_fraction_convergent_denominator(coefficients, n)), Rat.from_nat(continued_fraction_convergent_numerator(coefficients, n)) / Rat.from_nat(continued_fraction_convergent_denominator(coefficients, n)))
        Real.from_rat(Rat.from_nat(continued_fraction_convergent_denominator(coefficients, n))) * Real.from_rat(Rat.from_nat(continued_fraction_convergent_numerator(coefficients, n)) / Rat.from_nat(continued_fraction_convergent_denominator(coefficients, n))) = Real.from_rat(Rat.from_nat(continued_fraction_convergent_denominator(coefficients, n)) * (Rat.from_nat(continued_fraction_convergent_numerator(coefficients, n)) / Rat.from_nat(continued_fraction_convergent_denominator(coefficients, n))))
        Rat.from_nat(continued_fraction_convergent_denominator(coefficients, n)) * (Rat.from_nat(continued_fraction_convergent_numerator(coefficients, n)) / Rat.from_nat(continued_fraction_convergent_denominator(coefficients, n))) = Rat.from_nat(continued_fraction_convergent_numerator(coefficients, n))
        Real.from_rat(Rat.from_nat(continued_fraction_convergent_denominator(coefficients, n))) * Real.from_rat(Rat.from_nat(continued_fraction_convergent_numerator(coefficients, n)) / Rat.from_nat(continued_fraction_convergent_denominator(coefficients, n))) = Real.from_rat(Rat.from_nat(continued_fraction_convergent_numerator(coefficients, n)))
        from_nat[Real](continued_fraction_convergent_denominator(coefficients, n)) * continued_fraction_real_convergent_value(coefficients, n) = from_nat[Real](continued_fraction_convergent_numerator(coefficients, n))
        mul_sub_distrib_right(from_nat[Real](continued_fraction_convergent_denominator(coefficients, n)), continued_fraction_real_limit(coefficients), continued_fraction_real_convergent_value(coefficients, n))
        from_nat[Real](continued_fraction_convergent_denominator(coefficients, n)) * (continued_fraction_real_limit(coefficients) - continued_fraction_real_convergent_value(coefficients, n)) = from_nat[Real](continued_fraction_convergent_denominator(coefficients, n)) * continued_fraction_real_limit(coefficients) - from_nat[Real](continued_fraction_convergent_denominator(coefficients, n)) * continued_fraction_real_convergent_value(coefficients, n)
        from_nat[Real](continued_fraction_convergent_denominator(coefficients, n)) * (continued_fraction_real_limit(coefficients) - continued_fraction_real_convergent_value(coefficients, n)) = from_nat[Real](continued_fraction_convergent_denominator(coefficients, n)) * continued_fraction_real_limit(coefficients) - from_nat[Real](continued_fraction_convergent_numerator(coefficients, n))
        nat_lt_imp_rat_lt(Nat.0, continued_fraction_convergent_denominator(coefficients, n))
        Rat.from_nat(Nat.0) < Rat.from_nat(continued_fraction_convergent_denominator(coefficients, n))
        from_nat_zero[Rat]
        Rat.from_nat(Nat.0) = Rat.0
        Rat.0 < Rat.from_nat(continued_fraction_convergent_denominator(coefficients, n))
        from_rat_maintains_lt(Rat.0, Rat.from_nat(continued_fraction_convergent_denominator(coefficients, n)))
        Real.from_rat(Rat.0) < Real.from_rat(Rat.from_nat(continued_fraction_convergent_denominator(coefficients, n)))
        Real.from_rat(Rat.0) = Real.0
        Real.0 < Real.from_rat(Rat.from_nat(continued_fraction_convergent_denominator(coefficients, n)))
        Real.from_rat(Rat.from_nat(continued_fraction_convergent_denominator(coefficients, n))) = from_nat[Real](continued_fraction_convergent_denominator(coefficients, n))
        Real.0 < from_nat[Real](continued_fraction_convergent_denominator(coefficients, n))
        gt_zero_imp_pos(from_nat[Real](continued_fraction_convergent_denominator(coefficients, n)))
        from_nat[Real](continued_fraction_convergent_denominator(coefficients, n)).is_positive
        pos_imp_eq_abs(from_nat[Real](continued_fraction_convergent_denominator(coefficients, n)))
        from_nat[Real](continued_fraction_convergent_denominator(coefficients, n)).abs = from_nat[Real](continued_fraction_convergent_denominator(coefficients, n))
        mul_abs(from_nat[Real](continued_fraction_convergent_denominator(coefficients, n)), continued_fraction_real_limit(coefficients) - continued_fraction_real_convergent_value(coefficients, n))
        from_nat[Real](continued_fraction_convergent_denominator(coefficients, n)).abs * (continued_fraction_real_limit(coefficients) - continued_fraction_real_convergent_value(coefficients, n)).abs = (from_nat[Real](continued_fraction_convergent_denominator(coefficients, n)) * (continued_fraction_real_limit(coefficients) - continued_fraction_real_convergent_value(coefficients, n))).abs
        from_nat[Real](continued_fraction_convergent_denominator(coefficients, n)) * (continued_fraction_real_limit(coefficients) - continued_fraction_real_convergent_value(coefficients, n)).abs = (from_nat[Real](continued_fraction_convergent_denominator(coefficients, n)) * (continued_fraction_real_limit(coefficients) - continued_fraction_real_convergent_value(coefficients, n))).abs
        (from_nat[Real](continued_fraction_convergent_denominator(coefficients, n)) * continued_fraction_real_limit(coefficients) - from_nat[Real](continued_fraction_convergent_numerator(coefficients, n))).abs = from_nat[Real](continued_fraction_convergent_denominator(coefficients, n)) * (continued_fraction_real_limit(coefficients) - continued_fraction_real_convergent_value(coefficients, n)).abs
        lt_mul_pos_left((continued_fraction_real_limit(coefficients) - continued_fraction_real_convergent_value(coefficients, n)).abs, Real.from_rat(Rat.1 / Rat.from_nat(continued_fraction_convergent_denominator(coefficients, n) * continued_fraction_convergent_denominator(coefficients, n.suc))), from_nat[Real](continued_fraction_convergent_denominator(coefficients, n)))
        from_nat[Real](continued_fraction_convergent_denominator(coefficients, n)) * (continued_fraction_real_limit(coefficients) - continued_fraction_real_convergent_value(coefficients, n)).abs < from_nat[Real](continued_fraction_convergent_denominator(coefficients, n)) * Real.from_rat(Rat.1 / Rat.from_nat(continued_fraction_convergent_denominator(coefficients, n) * continued_fraction_convergent_denominator(coefficients, n.suc)))
        (from_nat[Real](continued_fraction_convergent_denominator(coefficients, n)) * continued_fraction_real_limit(coefficients) - from_nat[Real](continued_fraction_convergent_numerator(coefficients, n))).abs < from_nat[Real](continued_fraction_convergent_denominator(coefficients, n)) * Real.from_rat(Rat.1 / Rat.from_nat(continued_fraction_convergent_denominator(coefficients, n) * continued_fraction_convergent_denominator(coefficients, n.suc)))
        mul_from_rat(Rat.from_nat(continued_fraction_convergent_denominator(coefficients, n)), Rat.1 / Rat.from_nat(continued_fraction_convergent_denominator(coefficients, n) * continued_fraction_convergent_denominator(coefficients, n.suc)))
        Real.from_rat(Rat.from_nat(continued_fraction_convergent_denominator(coefficients, n))) * Real.from_rat(Rat.1 / Rat.from_nat(continued_fraction_convergent_denominator(coefficients, n) * continued_fraction_convergent_denominator(coefficients, n.suc))) = Real.from_rat(Rat.from_nat(continued_fraction_convergent_denominator(coefficients, n)) * (Rat.1 / Rat.from_nat(continued_fraction_convergent_denominator(coefficients, n) * continued_fraction_convergent_denominator(coefficients, n.suc))))
        from_nat_mul(continued_fraction_convergent_denominator(coefficients, n), continued_fraction_convergent_denominator(coefficients, n.suc))
        Rat.from_nat(continued_fraction_convergent_denominator(coefficients, n) * continued_fraction_convergent_denominator(coefficients, n.suc)) = Rat.from_nat(continued_fraction_convergent_denominator(coefficients, n)) * Rat.from_nat(continued_fraction_convergent_denominator(coefficients, n.suc))
        Rat.from_nat(continued_fraction_convergent_denominator(coefficients, n)) * (Rat.1 / Rat.from_nat(continued_fraction_convergent_denominator(coefficients, n) * continued_fraction_convergent_denominator(coefficients, n.suc))) = Rat.from_nat(continued_fraction_convergent_denominator(coefficients, n)) * (Rat.1 / (Rat.from_nat(continued_fraction_convergent_denominator(coefficients, n)) * Rat.from_nat(continued_fraction_convergent_denominator(coefficients, n.suc))))
        recip_eq_one_div(Rat.from_nat(continued_fraction_convergent_denominator(coefficients, n)) * Rat.from_nat(continued_fraction_convergent_denominator(coefficients, n.suc)))
        (Rat.from_nat(continued_fraction_convergent_denominator(coefficients, n)) * Rat.from_nat(continued_fraction_convergent_denominator(coefficients, n.suc))).inverse = Rat.1 / (Rat.from_nat(continued_fraction_convergent_denominator(coefficients, n)) * Rat.from_nat(continued_fraction_convergent_denominator(coefficients, n.suc)))
        recip_mul(Rat.from_nat(continued_fraction_convergent_denominator(coefficients, n)), Rat.from_nat(continued_fraction_convergent_denominator(coefficients, n.suc)))
        (Rat.from_nat(continued_fraction_convergent_denominator(coefficients, n)) * Rat.from_nat(continued_fraction_convergent_denominator(coefficients, n.suc))).inverse = Rat.from_nat(continued_fraction_convergent_denominator(coefficients, n)).inverse * Rat.from_nat(continued_fraction_convergent_denominator(coefficients, n.suc)).inverse
        Rat.1 / (Rat.from_nat(continued_fraction_convergent_denominator(coefficients, n)) * Rat.from_nat(continued_fraction_convergent_denominator(coefficients, n.suc))) = Rat.from_nat(continued_fraction_convergent_denominator(coefficients, n)).inverse * Rat.from_nat(continued_fraction_convergent_denominator(coefficients, n.suc)).inverse
        mul_inv_cancels_right(Rat.from_nat(continued_fraction_convergent_denominator(coefficients, n)))
        Rat.from_nat(continued_fraction_convergent_denominator(coefficients, n)) * Rat.from_nat(continued_fraction_convergent_denominator(coefficients, n)).inverse = Rat.1
        Rat.from_nat(continued_fraction_convergent_denominator(coefficients, n)) * (Rat.from_nat(continued_fraction_convergent_denominator(coefficients, n)).inverse * Rat.from_nat(continued_fraction_convergent_denominator(coefficients, n.suc)).inverse) = Rat.1 * Rat.from_nat(continued_fraction_convergent_denominator(coefficients, n.suc)).inverse
        Rat.1 * Rat.from_nat(continued_fraction_convergent_denominator(coefficients, n.suc)).inverse = Rat.from_nat(continued_fraction_convergent_denominator(coefficients, n.suc)).inverse
        Rat.from_nat(continued_fraction_convergent_denominator(coefficients, n)) * (Rat.from_nat(continued_fraction_convergent_denominator(coefficients, n)).inverse * Rat.from_nat(continued_fraction_convergent_denominator(coefficients, n.suc)).inverse) = Rat.from_nat(continued_fraction_convergent_denominator(coefficients, n.suc)).inverse
        Rat.from_nat(continued_fraction_convergent_denominator(coefficients, n)) * (Rat.1 / (Rat.from_nat(continued_fraction_convergent_denominator(coefficients, n)) * Rat.from_nat(continued_fraction_convergent_denominator(coefficients, n.suc)))) = Rat.from_nat(continued_fraction_convergent_denominator(coefficients, n.suc)).inverse
        recip_eq_one_div(Rat.from_nat(continued_fraction_convergent_denominator(coefficients, n.suc)))
        Rat.from_nat(continued_fraction_convergent_denominator(coefficients, n.suc)).inverse = Rat.1 / Rat.from_nat(continued_fraction_convergent_denominator(coefficients, n.suc))
        Rat.from_nat(continued_fraction_convergent_denominator(coefficients, n)) * (Rat.1 / (Rat.from_nat(continued_fraction_convergent_denominator(coefficients, n)) * Rat.from_nat(continued_fraction_convergent_denominator(coefficients, n.suc)))) = Rat.1 / Rat.from_nat(continued_fraction_convergent_denominator(coefficients, n.suc))
        Rat.from_nat(continued_fraction_convergent_denominator(coefficients, n)) * (Rat.1 / Rat.from_nat(continued_fraction_convergent_denominator(coefficients, n) * continued_fraction_convergent_denominator(coefficients, n.suc))) = Rat.1 / Rat.from_nat(continued_fraction_convergent_denominator(coefficients, n.suc))
        Real.from_rat(Rat.from_nat(continued_fraction_convergent_denominator(coefficients, n)) * (Rat.1 / Rat.from_nat(continued_fraction_convergent_denominator(coefficients, n) * continued_fraction_convergent_denominator(coefficients, n.suc)))) = Real.from_rat(Rat.1 / Rat.from_nat(continued_fraction_convergent_denominator(coefficients, n.suc)))
        Real.from_rat(Rat.from_nat(continued_fraction_convergent_denominator(coefficients, n))) * Real.from_rat(Rat.1 / Rat.from_nat(continued_fraction_convergent_denominator(coefficients, n) * continued_fraction_convergent_denominator(coefficients, n.suc))) = Real.from_rat(Rat.1 / Rat.from_nat(continued_fraction_convergent_denominator(coefficients, n.suc)))
        from_nat[Real](continued_fraction_convergent_denominator(coefficients, n)) * Real.from_rat(Rat.1 / Rat.from_nat(continued_fraction_convergent_denominator(coefficients, n) * continued_fraction_convergent_denominator(coefficients, n.suc))) = Real.from_rat(Rat.1 / Rat.from_nat(continued_fraction_convergent_denominator(coefficients, n.suc)))
        (from_nat[Real](continued_fraction_convergent_denominator(coefficients, n)) * continued_fraction_real_limit(coefficients) - from_nat[Real](continued_fraction_convergent_numerator(coefficients, n))).abs < Real.from_rat(Rat.1 / Rat.from_nat(continued_fraction_convergent_denominator(coefficients, n.suc)))
        lt_suc(Nat.0)
        Nat.0 < Nat.1
        lt_and_lte(Nat.0, Nat.1, bound)
        Nat.0 < bound
        nat_lt_imp_rat_lt(Nat.0, bound)
        Rat.from_nat(Nat.0) < Rat.from_nat(bound)
        Rat.from_nat(Nat.0) = Rat.0
        Rat.0 < Rat.from_nat(bound)
        from_rat_maintains_lt(Rat.0, Rat.from_nat(bound))
        Real.from_rat(Rat.0) < Real.from_rat(Rat.from_nat(bound))
        Real.from_rat(Rat.0) = Real.0
        Real.0 < Real.from_rat(Rat.from_nat(bound))
        from_nat_is_from_rat(bound)
        from_nat[Real](bound) = Real.from_rat(Rat.from_nat(bound))
        Real.0 < from_nat[Real](bound)
        nat_lt_imp_rat_lt(Nat.0, continued_fraction_convergent_denominator(coefficients, n.suc))
        Rat.from_nat(Nat.0) < Rat.from_nat(continued_fraction_convergent_denominator(coefficients, n.suc))
        Rat.from_nat(Nat.0) = Rat.0
        Rat.0 < Rat.from_nat(continued_fraction_convergent_denominator(coefficients, n.suc))
        from_rat_maintains_lt(Rat.0, Rat.from_nat(continued_fraction_convergent_denominator(coefficients, n.suc)))
        Real.from_rat(Rat.0) < Real.from_rat(Rat.from_nat(continued_fraction_convergent_denominator(coefficients, n.suc)))
        Real.from_rat(Rat.0) = Real.0
        Real.0 < Real.from_rat(Rat.from_nat(continued_fraction_convergent_denominator(coefficients, n.suc)))
        from_nat_is_from_rat(continued_fraction_convergent_denominator(coefficients, n.suc))
        from_nat[Real](continued_fraction_convergent_denominator(coefficients, n.suc)) = Real.from_rat(Rat.from_nat(continued_fraction_convergent_denominator(coefficients, n.suc)))
        Real.0 < from_nat[Real](continued_fraction_convergent_denominator(coefficients, n.suc))
        nat_lt_imp_rat_lt(bound, continued_fraction_convergent_denominator(coefficients, n.suc))
        Rat.from_nat(bound) < Rat.from_nat(continued_fraction_convergent_denominator(coefficients, n.suc))
        from_rat_maintains_lt(Rat.from_nat(bound), Rat.from_nat(continued_fraction_convergent_denominator(coefficients, n.suc)))
        Real.from_rat(Rat.from_nat(bound)) < Real.from_rat(Rat.from_nat(continued_fraction_convergent_denominator(coefficients, n.suc)))
        from_nat[Real](bound) < from_nat[Real](continued_fraction_convergent_denominator(coefficients, n.suc))
        inverse_on_positive_flips_inequality(from_nat[Real](bound), from_nat[Real](continued_fraction_convergent_denominator(coefficients, n.suc)))
        from_nat[Real](continued_fraction_convergent_denominator(coefficients, n.suc)).inverse < from_nat[Real](bound).inverse
        real_from_nat_recip_inverse(continued_fraction_convergent_denominator(coefficients, n.suc))
        Real.from_rat(Rat.1 / Rat.from_nat(continued_fraction_convergent_denominator(coefficients, n.suc))) = Real.from_rat(Rat.from_nat(continued_fraction_convergent_denominator(coefficients, n.suc))).inverse
        Real.from_rat(Rat.from_nat(continued_fraction_convergent_denominator(coefficients, n.suc))) = from_nat[Real](continued_fraction_convergent_denominator(coefficients, n.suc))
        Real.from_rat(Rat.1 / Rat.from_nat(continued_fraction_convergent_denominator(coefficients, n.suc))) = from_nat[Real](continued_fraction_convergent_denominator(coefficients, n.suc)).inverse
        real_from_nat_recip_inverse(bound)
        Real.from_rat(Rat.1 / Rat.from_nat(bound)) = Real.from_rat(Rat.from_nat(bound)).inverse
        Real.from_rat(Rat.from_nat(bound)) = from_nat[Real](bound)
        Real.from_rat(Rat.1 / Rat.from_nat(bound)) = from_nat[Real](bound).inverse
        Real.from_rat(Rat.1 / Rat.from_nat(continued_fraction_convergent_denominator(coefficients, n.suc))) < Real.from_rat(Rat.1 / Rat.from_nat(bound))
        lt_trans((from_nat[Real](continued_fraction_convergent_denominator(coefficients, n)) * continued_fraction_real_limit(coefficients) - from_nat[Real](continued_fraction_convergent_numerator(coefficients, n))).abs, Real.from_rat(Rat.1 / Rat.from_nat(continued_fraction_convergent_denominator(coefficients, n.suc))), Real.from_rat(Rat.1 / Rat.from_nat(bound)))
        (from_nat[Real](continued_fraction_convergent_denominator(coefficients, n)) * continued_fraction_real_limit(coefficients) - from_nat[Real](continued_fraction_convergent_numerator(coefficients, n))).abs < Real.from_rat(Rat.1 / Rat.from_nat(bound))
        Nat.1 <= continued_fraction_convergent_denominator(coefficients, n) and continued_fraction_convergent_denominator(coefficients, n) <= bound and (from_nat[Real](continued_fraction_convergent_denominator(coefficients, n)) * continued_fraction_real_limit(coefficients) - from_nat[Real](continued_fraction_convergent_numerator(coefficients, n))).abs < Real.from_rat(Rat.1 / Rat.from_nat(bound))
        dirichlet_good_approximation(continued_fraction_real_limit(coefficients), continued_fraction_convergent_numerator(coefficients, n), continued_fraction_convergent_denominator(coefficients, n), bound) = Nat.1 <= continued_fraction_convergent_denominator(coefficients, n) and continued_fraction_convergent_denominator(coefficients, n) <= bound and (from_nat[Real](continued_fraction_convergent_denominator(coefficients, n)) * continued_fraction_real_limit(coefficients) - from_nat[Real](continued_fraction_convergent_numerator(coefficients, n))).abs < Real.from_rat(Rat.1 / Rat.from_nat(bound))
        dirichlet_good_approximation(continued_fraction_real_limit(coefficients), continued_fraction_convergent_numerator(coefficients, n), continued_fraction_convergent_denominator(coefficients, n), bound)
        exists(p: Nat, q: Nat) {
            dirichlet_good_approximation(continued_fraction_real_limit(coefficients), p, q, bound)
        }
    }
}

// ============================================================================
// Section 4: the general Dirichlet theorem and the irrationality measure
// ============================================================================

// Dirichlet's approximation theorem for an arbitrary real alpha.  The library
// does not yet construct the continued-fraction expansion of an arbitrary real
// number (there is no floor/expansion operation on `Real`), so the theorem
// cannot be proved from the convergents in full generality; the convergent
// version `diophantine_dirichlet` proved in Section 3 is the form the
// library's machinery supports.
//
// theorem dirichlet_approximation_theorem(alpha: Real, bound: Nat) {
//     Nat.1 <= bound implies exists(p: Nat, q: Nat) {
//         Nat.1 <= q and q <= bound and
//         (from_nat[Real](q) * alpha - from_nat[Real](p)).abs <
//             Real.from_rat(Rat.1 / Rat.from_nat(bound))
//     }
// }

// The irrationality measure: the converse of Section 2.  If
// `|alpha - p / q| < 1 / q^2` holds for infinitely many reduced `p / q`, then
// alpha is irrational.
//
// Without the reduced hypothesis the statement is false: a rational
// `alpha = a / b` has the infinitely many (non-reduced) approximations
// `p / q = (k * a) / (k * b)` with `|alpha - p / q| = 0 < 1 / q^2`.
//
// The proof is the standard denominator bound.  For a rational
// `alpha = a / b` and a reduced `p / q` with `|alpha - p / q| < 1 / q^2`:
//
//   `|a / b - p / q| = |a * q - b * p| / (b * q)`.
//
// If `a * q != b * p` then the integer `a * q - b * p` is nonzero, so the
// numerator is at least one and `1 / (b * q) < 1 / q^2`, which forces
// `q < b`.  If `a * q = b * p` then `p / q = a / b`; since `p` and `q` are
// coprime, Euclid's lemma gives `q | b`, so `q <= b`.  Either way `q` is
// bounded, contradicting an infinite family of approximations.  The library
// does not yet expose two ingredients this needs: a strict real-to-rational
// reflection lemma (`Real.from_rat(x) < Real.from_rat(y)` implies `x < y`),
// and a nonzero-integer absolute value bound (a nonzero `Int` embeds as a
// rational with absolute value at least one), so the theorem is recorded
// below for future work.
//
// theorem good_approximations_imp_irrational(alpha: Real) {
//     (forall(bound: Nat) {
//         exists(p: Nat, q: Nat) {
//             Nat.1 <= q and bound < q and q.coprime(p) and
//             (alpha - Real.from_rat(Rat.from_nat(p) / Rat.from_nat(q))).abs <
//                 Real.from_rat(Rat.1 / Rat.from_nat(q * q))
//         }
//     }) implies not exists(a: Nat, b: Nat) {
//         Nat.1 <= b and alpha = Real.from_rat(Rat.from_nat(a) / Rat.from_nat(b))
//     }
// }
