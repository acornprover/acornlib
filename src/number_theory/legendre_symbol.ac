from int import Int
from number_theory.quadratic_residue import Nat, is_quadratic_residue_mod,
    is_unit_quadratic_residue_mod, unit_quadratic_residue_is_residue,
    quadratic_residue_coprime_is_unit, prime_nonzero_congr_mod_imp_coprime,
    prime_coprime_imp_nonzero_congr_mod, prime_unit_quadratic_residue_nonzero
from number_theory.congruence import congr_mod_symm, congr_mod_trans,
    mod_congr_mod_self
from number_theory.quadratic_residue import quadratic_residue_congr_mod
numerals Nat
numerals Int

/// True when `a` is not a square modulo `n`.
define is_quadratic_nonresidue_mod(a: Nat, n: Nat) -> Bool {
    not is_quadratic_residue_mod(a, n)
}

/// True when `a` is a unit and not a square modulo `n`.
define is_unit_quadratic_nonresidue_mod(a: Nat, n: Nat) -> Bool {
    a.coprime(n) and is_quadratic_nonresidue_mod(a, n)
}

/// A quadratic nonresidue is not a quadratic residue.
theorem quadratic_nonresidue_not_residue(a: Nat, n: Nat) {
    is_quadratic_nonresidue_mod(a, n) implies not is_quadratic_residue_mod(a, n)
} by {
    if is_quadratic_nonresidue_mod(a, n) {
        is_quadratic_nonresidue_mod(a, n) = not is_quadratic_residue_mod(a, n)
        not is_quadratic_residue_mod(a, n)
    }
}

/// A residue that is not quadratic is a quadratic nonresidue.
theorem quadratic_nonresidue_of_not_residue(a: Nat, n: Nat) {
    not is_quadratic_residue_mod(a, n) implies is_quadratic_nonresidue_mod(a, n)
} by {
    if not is_quadratic_residue_mod(a, n) {
        is_quadratic_nonresidue_mod(a, n) = not is_quadratic_residue_mod(a, n)
        is_quadratic_nonresidue_mod(a, n)
    }
}

/// Congruent targets preserve quadratic nonresidue status.
theorem quadratic_nonresidue_congr_mod(a: Nat, b: Nat, n: Nat) {
    is_quadratic_nonresidue_mod(a, n) and a.congr_mod(b, n)
        implies is_quadratic_nonresidue_mod(b, n)
} by {
    if is_quadratic_nonresidue_mod(a, n) and a.congr_mod(b, n) {
        quadratic_nonresidue_not_residue(a, n)
        if is_quadratic_residue_mod(b, n) {
            congr_mod_symm(a, b, n)
            b.congr_mod(a, n)
            quadratic_residue_congr_mod(b, a, n)
            is_quadratic_residue_mod(a, n)
            false
        }
        quadratic_nonresidue_of_not_residue(b, n)
        is_quadratic_nonresidue_mod(b, n)
    }
}

/// A unit quadratic nonresidue is a unit.
theorem unit_quadratic_nonresidue_coprime(a: Nat, n: Nat) {
    is_unit_quadratic_nonresidue_mod(a, n) implies a.coprime(n)
} by {
    if is_unit_quadratic_nonresidue_mod(a, n) {
        is_unit_quadratic_nonresidue_mod(a, n) =
            (a.coprime(n) and is_quadratic_nonresidue_mod(a, n))
        a.coprime(n)
    }
}

/// A unit quadratic nonresidue is a quadratic nonresidue.
theorem unit_quadratic_nonresidue_is_nonresidue(a: Nat, n: Nat) {
    is_unit_quadratic_nonresidue_mod(a, n) implies is_quadratic_nonresidue_mod(a, n)
} by {
    if is_unit_quadratic_nonresidue_mod(a, n) {
        is_unit_quadratic_nonresidue_mod(a, n) =
            (a.coprime(n) and is_quadratic_nonresidue_mod(a, n))
        is_quadratic_nonresidue_mod(a, n)
    }
}

/// A coprime quadratic nonresidue is a unit quadratic nonresidue.
theorem quadratic_nonresidue_coprime_is_unit(a: Nat, n: Nat) {
    is_quadratic_nonresidue_mod(a, n) and a.coprime(n)
        implies is_unit_quadratic_nonresidue_mod(a, n)
} by {
    if is_quadratic_nonresidue_mod(a, n) and a.coprime(n) {
        is_unit_quadratic_nonresidue_mod(a, n) =
            (a.coprime(n) and is_quadratic_nonresidue_mod(a, n))
        is_unit_quadratic_nonresidue_mod(a, n)
    }
}

/// Unit and ordinary nonresidue predicates agree on coprime targets.
theorem quadratic_nonresidue_coprime_iff_unit(a: Nat, n: Nat) {
    a.coprime(n) implies
        is_quadratic_nonresidue_mod(a, n) = is_unit_quadratic_nonresidue_mod(a, n)
} by {
    if a.coprime(n) {
        if is_quadratic_nonresidue_mod(a, n) {
            quadratic_nonresidue_coprime_is_unit(a, n)
            is_unit_quadratic_nonresidue_mod(a, n)
        }
        if is_unit_quadratic_nonresidue_mod(a, n) {
            unit_quadratic_nonresidue_is_nonresidue(a, n)
            is_quadratic_nonresidue_mod(a, n)
        }
        (is_quadratic_nonresidue_mod(a, n) =
            is_unit_quadratic_nonresidue_mod(a, n)) = true
    }
}

/// True when an integer is one of the three possible Legendre-symbol values.
define is_legendre_symbol_value(s: Int) -> Bool {
    s = Int.0 or s = Int.1 or s = -Int.1
}

/// True when the Legendre symbol should take the value `1`.
define legendre_symbol_one_case(a: Nat, p: Nat) -> Bool {
    not a.congr_mod(Nat.0, p) and is_quadratic_residue_mod(a, p)
}

/// True when the Legendre symbol should take the value `0`.
define legendre_symbol_zero_case(a: Nat, p: Nat) -> Bool {
    a.congr_mod(Nat.0, p)
}

/// True when the Legendre symbol should take the value `-1`.
define legendre_symbol_neg_one_case(a: Nat, p: Nat) -> Bool {
    not a.congr_mod(Nat.0, p) and is_quadratic_nonresidue_mod(a, p)
}

/// The integer-valued Legendre symbol. It is `0` on the zero residue class,
/// `1` on nonzero quadratic residues, and `-1` on quadratic nonresidues.
define legendre_symbol(a: Nat, p: Nat) -> Int {
    if legendre_symbol_zero_case(a, p) {
        Int.0
    } else {
        if legendre_symbol_one_case(a, p) {
            Int.1
        } else {
            -Int.1
        }
    }
}

/// Nonzero quadratic residues are exactly the `1` case.
theorem legendre_symbol_one_case_of_nonzero_residue(a: Nat, p: Nat) {
    not a.congr_mod(Nat.0, p) and is_quadratic_residue_mod(a, p)
        implies legendre_symbol_one_case(a, p)
} by {
    if not a.congr_mod(Nat.0, p) and is_quadratic_residue_mod(a, p) {
        legendre_symbol_one_case(a, p) =
            (not a.congr_mod(Nat.0, p) and is_quadratic_residue_mod(a, p))
        legendre_symbol_one_case(a, p)
    }
}

/// Zero residues are exactly the `0` case.
theorem legendre_symbol_zero_case_of_congr_zero(a: Nat, p: Nat) {
    a.congr_mod(Nat.0, p) implies legendre_symbol_zero_case(a, p)
} by {
    if a.congr_mod(Nat.0, p) {
        legendre_symbol_zero_case(a, p) = a.congr_mod(Nat.0, p)
        legendre_symbol_zero_case(a, p)
    }
}

/// Nonzero quadratic nonresidues are exactly the `-1` case.
theorem legendre_symbol_neg_one_case_of_nonzero_nonresidue(a: Nat, p: Nat) {
    not a.congr_mod(Nat.0, p) and is_quadratic_nonresidue_mod(a, p)
        implies legendre_symbol_neg_one_case(a, p)
} by {
    if not a.congr_mod(Nat.0, p) and is_quadratic_nonresidue_mod(a, p) {
        legendre_symbol_neg_one_case(a, p) =
            (not a.congr_mod(Nat.0, p) and is_quadratic_nonresidue_mod(a, p))
        legendre_symbol_neg_one_case(a, p)
    }
}

/// Characterization of the `1` Legendre-symbol case.
theorem legendre_symbol_one_case_iff_nonzero_residue(a: Nat, p: Nat) {
    legendre_symbol_one_case(a, p) =
        (not a.congr_mod(Nat.0, p) and is_quadratic_residue_mod(a, p))
}

/// Characterization of the `0` Legendre-symbol case.
theorem legendre_symbol_zero_case_iff_congr_zero(a: Nat, p: Nat) {
    legendre_symbol_zero_case(a, p) = a.congr_mod(Nat.0, p)
}

/// Characterization of the `-1` Legendre-symbol case.
theorem legendre_symbol_neg_one_case_iff_nonzero_nonresidue(a: Nat, p: Nat) {
    legendre_symbol_neg_one_case(a, p) =
        (not a.congr_mod(Nat.0, p) and is_quadratic_nonresidue_mod(a, p))
}

/// Projection from the `1` case to nonzero status.
theorem legendre_symbol_one_case_nonzero(a: Nat, p: Nat) {
    legendre_symbol_one_case(a, p) implies not a.congr_mod(Nat.0, p)
} by {
    if legendre_symbol_one_case(a, p) {
        legendre_symbol_one_case_iff_nonzero_residue(a, p)
        not a.congr_mod(Nat.0, p)
    }
}

/// Projection from the `1` case to quadratic-residue status.
theorem legendre_symbol_one_case_residue(a: Nat, p: Nat) {
    legendre_symbol_one_case(a, p) implies is_quadratic_residue_mod(a, p)
} by {
    if legendre_symbol_one_case(a, p) {
        legendre_symbol_one_case_iff_nonzero_residue(a, p)
        is_quadratic_residue_mod(a, p)
    }
}

/// Projection from the `0` case to congruence with zero.
theorem legendre_symbol_zero_case_congr_zero(a: Nat, p: Nat) {
    legendre_symbol_zero_case(a, p) implies a.congr_mod(Nat.0, p)
} by {
    if legendre_symbol_zero_case(a, p) {
        legendre_symbol_zero_case_iff_congr_zero(a, p)
        a.congr_mod(Nat.0, p)
    }
}

/// The zero case is invariant under congruence.
theorem legendre_symbol_zero_case_congr_mod(a: Nat, b: Nat, p: Nat) {
    legendre_symbol_zero_case(a, p) and a.congr_mod(b, p)
        implies legendre_symbol_zero_case(b, p)
} by {
    if legendre_symbol_zero_case(a, p) and a.congr_mod(b, p) {
        legendre_symbol_zero_case(a, p) = a.congr_mod(Nat.0, p)
        a.congr_mod(Nat.0, p)
        congr_mod_symm(a, b, p)
        b.congr_mod(a, p)
        congr_mod_trans(b, a, Nat.0, p)
        b.congr_mod(Nat.0, p)
        legendre_symbol_zero_case(b, p) = b.congr_mod(Nat.0, p)
        legendre_symbol_zero_case(b, p)
    }
}

/// Congruent targets have equivalent zero Legendre cases.
theorem legendre_symbol_zero_case_congr_mod_iff(a: Nat, b: Nat, p: Nat) {
    a.congr_mod(b, p) implies
        legendre_symbol_zero_case(a, p) = legendre_symbol_zero_case(b, p)
} by {
    if a.congr_mod(b, p) {
        if legendre_symbol_zero_case(a, p) {
            legendre_symbol_zero_case_congr_mod(a, b, p)
            legendre_symbol_zero_case(b, p)
        }
        if legendre_symbol_zero_case(b, p) {
            congr_mod_symm(a, b, p)
            b.congr_mod(a, p)
            legendre_symbol_zero_case_congr_mod(b, a, p)
            legendre_symbol_zero_case(a, p)
        }
        (legendre_symbol_zero_case(a, p) = legendre_symbol_zero_case(b, p)) = true
    }
}

/// The one case is invariant under congruence.
theorem legendre_symbol_one_case_congr_mod(a: Nat, b: Nat, p: Nat) {
    legendre_symbol_one_case(a, p) and a.congr_mod(b, p)
        implies legendre_symbol_one_case(b, p)
} by {
    if legendre_symbol_one_case(a, p) and a.congr_mod(b, p) {
        legendre_symbol_one_case(a, p) =
            (not a.congr_mod(Nat.0, p) and is_quadratic_residue_mod(a, p))
        not a.congr_mod(Nat.0, p)
        is_quadratic_residue_mod(a, p)
        if b.congr_mod(Nat.0, p) {
            congr_mod_trans(a, b, Nat.0, p)
            a.congr_mod(Nat.0, p)
            false
        }
        quadratic_residue_congr_mod(a, b, p)
        is_quadratic_residue_mod(b, p)
        legendre_symbol_one_case(b, p) =
            (not b.congr_mod(Nat.0, p) and is_quadratic_residue_mod(b, p))
        legendre_symbol_one_case(b, p)
    }
}

/// Congruent targets have equivalent `1` Legendre cases.
theorem legendre_symbol_one_case_congr_mod_iff(a: Nat, b: Nat, p: Nat) {
    a.congr_mod(b, p) implies
        legendre_symbol_one_case(a, p) = legendre_symbol_one_case(b, p)
} by {
    if a.congr_mod(b, p) {
        if legendre_symbol_one_case(a, p) {
            legendre_symbol_one_case_congr_mod(a, b, p)
            legendre_symbol_one_case(b, p)
        }
        if legendre_symbol_one_case(b, p) {
            congr_mod_symm(a, b, p)
            b.congr_mod(a, p)
            legendre_symbol_one_case_congr_mod(b, a, p)
            legendre_symbol_one_case(a, p)
        }
        (legendre_symbol_one_case(a, p) = legendre_symbol_one_case(b, p)) = true
    }
}

/// The one case excludes the zero case.
theorem legendre_symbol_one_case_not_zero_case(a: Nat, p: Nat) {
    legendre_symbol_one_case(a, p) implies not legendre_symbol_zero_case(a, p)
} by {
    if legendre_symbol_one_case(a, p) {
        legendre_symbol_one_case(a, p) =
            (not a.congr_mod(Nat.0, p) and is_quadratic_residue_mod(a, p))
        not a.congr_mod(Nat.0, p)
        if legendre_symbol_zero_case(a, p) {
            legendre_symbol_zero_case(a, p) = a.congr_mod(Nat.0, p)
            a.congr_mod(Nat.0, p)
            false
        }
    }
}

/// The minus-one case excludes the zero case.
theorem legendre_symbol_neg_one_case_not_zero_case(a: Nat, p: Nat) {
    legendre_symbol_neg_one_case(a, p) implies not legendre_symbol_zero_case(a, p)
} by {
    if legendre_symbol_neg_one_case(a, p) {
        legendre_symbol_neg_one_case(a, p) =
            (not a.congr_mod(Nat.0, p) and is_quadratic_nonresidue_mod(a, p))
        not a.congr_mod(Nat.0, p)
        if legendre_symbol_zero_case(a, p) {
            legendre_symbol_zero_case(a, p) = a.congr_mod(Nat.0, p)
            a.congr_mod(Nat.0, p)
            false
        }
    }
}

/// Projection from the minus-one case to quadratic-nonresidue status.
theorem legendre_symbol_neg_one_case_nonresidue(a: Nat, p: Nat) {
    legendre_symbol_neg_one_case(a, p) implies is_quadratic_nonresidue_mod(a, p)
} by {
    if legendre_symbol_neg_one_case(a, p) {
        legendre_symbol_neg_one_case_iff_nonzero_nonresidue(a, p)
        is_quadratic_nonresidue_mod(a, p)
    }
}

/// The one case excludes the minus-one case.
theorem legendre_symbol_one_case_not_neg_one_case(a: Nat, p: Nat) {
    legendre_symbol_one_case(a, p) implies not legendre_symbol_neg_one_case(a, p)
} by {
    if legendre_symbol_one_case(a, p) {
        legendre_symbol_one_case(a, p) =
            (not a.congr_mod(Nat.0, p) and is_quadratic_residue_mod(a, p))
        is_quadratic_residue_mod(a, p)
        if legendre_symbol_neg_one_case(a, p) {
            legendre_symbol_neg_one_case(a, p) =
                (not a.congr_mod(Nat.0, p) and is_quadratic_nonresidue_mod(a, p))
            is_quadratic_nonresidue_mod(a, p)
            quadratic_nonresidue_not_residue(a, p)
            not is_quadratic_residue_mod(a, p)
            false
        }
    }
}

/// Every residue class falls into one of the three Legendre cases.
theorem legendre_symbol_case_exhaustive(a: Nat, p: Nat) {
    legendre_symbol_one_case(a, p) or legendre_symbol_zero_case(a, p) or
        legendre_symbol_neg_one_case(a, p)
} by {
    if a.congr_mod(Nat.0, p) {
        legendre_symbol_zero_case_of_congr_zero(a, p)
        legendre_symbol_zero_case(a, p)
        legendre_symbol_one_case(a, p) or legendre_symbol_zero_case(a, p) or
            legendre_symbol_neg_one_case(a, p)
    } else {
        if is_quadratic_residue_mod(a, p) {
            legendre_symbol_one_case_of_nonzero_residue(a, p)
            legendre_symbol_one_case(a, p)
            legendre_symbol_one_case(a, p) or legendre_symbol_zero_case(a, p) or
                legendre_symbol_neg_one_case(a, p)
        } else {
            quadratic_nonresidue_of_not_residue(a, p)
            is_quadratic_nonresidue_mod(a, p)
            legendre_symbol_neg_one_case_of_nonzero_nonresidue(a, p)
            legendre_symbol_neg_one_case(a, p)
            legendre_symbol_one_case(a, p) or legendre_symbol_zero_case(a, p) or
                legendre_symbol_neg_one_case(a, p)
        }
    }
}

/// The Legendre symbol is one on nonzero quadratic residues.
theorem legendre_symbol_one_of_nonzero_residue(a: Nat, p: Nat) {
    not a.congr_mod(Nat.0, p) and is_quadratic_residue_mod(a, p)
        implies legendre_symbol(a, p) = Int.1
} by {
    if not a.congr_mod(Nat.0, p) and is_quadratic_residue_mod(a, p) {
        legendre_symbol_one_case_of_nonzero_residue(a, p)
        legendre_symbol_one_case(a, p)
        if legendre_symbol_zero_case(a, p) {
            legendre_symbol_zero_case_congr_zero(a, p)
            a.congr_mod(Nat.0, p)
            false
        } else {
            let s: Int = if legendre_symbol_zero_case(a, p) {
                Int.0
            } else {
                if legendre_symbol_one_case(a, p) {
                    Int.1
                } else {
                    -Int.1
                }
            }
            s = legendre_symbol(a, p)
            s = Int.1
            legendre_symbol(a, p) = Int.1
        }
    }
}

/// A unit quadratic residue modulo a prime has Legendre symbol one.
theorem legendre_symbol_one_of_prime_unit_quadratic_residue(p: Nat, a: Nat) {
    p.is_prime and is_unit_quadratic_residue_mod(a, p)
        implies legendre_symbol(a, p) = Int.1
} by {
    if p.is_prime and is_unit_quadratic_residue_mod(a, p) {
        prime_unit_quadratic_residue_nonzero(p, a)
        not a.congr_mod(Nat.0, p)
        unit_quadratic_residue_is_residue(a, p)
        is_quadratic_residue_mod(a, p)
        legendre_symbol_one_of_nonzero_residue(a, p)
        legendre_symbol(a, p) = Int.1
    }
}

/// Over a prime modulus, the `1` Legendre case is a coprime residue class.
theorem legendre_symbol_one_case_coprime_prime(p: Nat, a: Nat) {
    p.is_prime and legendre_symbol_one_case(a, p) implies a.coprime(p)
} by {
    if p.is_prime and legendre_symbol_one_case(a, p) {
        legendre_symbol_one_case_nonzero(a, p)
        not a.congr_mod(Nat.0, p)
        prime_nonzero_congr_mod_imp_coprime(p, a)
        a.coprime(p)
    }
}

/// Over a prime modulus, the `1` Legendre case is a unit quadratic residue.
theorem legendre_symbol_one_case_unit_quadratic_residue_prime(p: Nat, a: Nat) {
    p.is_prime and legendre_symbol_one_case(a, p)
        implies is_unit_quadratic_residue_mod(a, p)
} by {
    if p.is_prime and legendre_symbol_one_case(a, p) {
        legendre_symbol_one_case_residue(a, p)
        is_quadratic_residue_mod(a, p)
        legendre_symbol_one_case_coprime_prime(p, a)
        a.coprime(p)
        quadratic_residue_coprime_is_unit(a, p)
        is_unit_quadratic_residue_mod(a, p)
    }
}

/// A unit quadratic residue modulo a prime is in the `1` Legendre case.
theorem legendre_symbol_one_case_of_prime_unit_quadratic_residue(p: Nat, a: Nat) {
    p.is_prime and is_unit_quadratic_residue_mod(a, p)
        implies legendre_symbol_one_case(a, p)
} by {
    if p.is_prime and is_unit_quadratic_residue_mod(a, p) {
        prime_unit_quadratic_residue_nonzero(p, a)
        not a.congr_mod(Nat.0, p)
        unit_quadratic_residue_is_residue(a, p)
        is_quadratic_residue_mod(a, p)
        legendre_symbol_one_case_of_nonzero_residue(a, p)
        legendre_symbol_one_case(a, p)
    }
}

/// Over a prime modulus, the `1` Legendre case is exactly unit
/// quadratic-residue status.
theorem legendre_symbol_one_case_iff_prime_unit_quadratic_residue(p: Nat, a: Nat) {
    p.is_prime implies
        (legendre_symbol_one_case(a, p) = is_unit_quadratic_residue_mod(a, p))
} by {
    if p.is_prime {
        if legendre_symbol_one_case(a, p) {
            legendre_symbol_one_case_unit_quadratic_residue_prime(p, a)
            is_unit_quadratic_residue_mod(a, p)
        }
        if is_unit_quadratic_residue_mod(a, p) {
            legendre_symbol_one_case_of_prime_unit_quadratic_residue(p, a)
            legendre_symbol_one_case(a, p)
        }
        (legendre_symbol_one_case(a, p) = is_unit_quadratic_residue_mod(a, p)) = true
    }
}

/// Over a prime modulus, the `0` Legendre case implies non-coprimality.
theorem legendre_symbol_zero_case_imp_not_coprime_prime(p: Nat, a: Nat) {
    p.is_prime and legendre_symbol_zero_case(a, p) implies not a.coprime(p)
} by {
    if p.is_prime and legendre_symbol_zero_case(a, p) {
        legendre_symbol_zero_case_congr_zero(a, p)
        a.congr_mod(Nat.0, p)
        if a.coprime(p) {
            prime_coprime_imp_nonzero_congr_mod(p, a)
            not a.congr_mod(Nat.0, p)
            false
        }
    }
}

/// Over a prime modulus, non-coprimality gives the `0` Legendre case.
theorem legendre_symbol_zero_case_of_not_coprime_prime(p: Nat, a: Nat) {
    p.is_prime and not a.coprime(p) implies legendre_symbol_zero_case(a, p)
} by {
    if p.is_prime and not a.coprime(p) {
        if not legendre_symbol_zero_case(a, p) {
            legendre_symbol_zero_case_iff_congr_zero(a, p)
            not a.congr_mod(Nat.0, p)
            prime_nonzero_congr_mod_imp_coprime(p, a)
            a.coprime(p)
            false
        }
    }
}

/// Over a prime modulus, the `0` Legendre case is exactly non-coprimality.
theorem legendre_symbol_zero_case_iff_not_coprime_prime(p: Nat, a: Nat) {
    p.is_prime implies (legendre_symbol_zero_case(a, p) = not a.coprime(p))
} by {
    if p.is_prime {
        if legendre_symbol_zero_case(a, p) {
            legendre_symbol_zero_case_imp_not_coprime_prime(p, a)
            not a.coprime(p)
        }
        if not a.coprime(p) {
            legendre_symbol_zero_case_of_not_coprime_prime(p, a)
            legendre_symbol_zero_case(a, p)
        }
        (legendre_symbol_zero_case(a, p) = not a.coprime(p)) = true
    }
}

/// Over a prime modulus, a coprime nonresidue is in the `-1` Legendre case.
theorem legendre_symbol_neg_one_case_of_prime_coprime_nonresidue(p: Nat, a: Nat) {
    p.is_prime and a.coprime(p) and is_quadratic_nonresidue_mod(a, p)
        implies legendre_symbol_neg_one_case(a, p)
} by {
    if p.is_prime and a.coprime(p) and is_quadratic_nonresidue_mod(a, p) {
        prime_coprime_imp_nonzero_congr_mod(p, a)
        not a.congr_mod(Nat.0, p)
        legendre_symbol_neg_one_case_of_nonzero_nonresidue(a, p)
        legendre_symbol_neg_one_case(a, p)
    }
}

/// Over a prime modulus, a unit nonresidue is in the `-1` Legendre case.
theorem legendre_symbol_neg_one_case_of_prime_unit_nonresidue(p: Nat, a: Nat) {
    p.is_prime and is_unit_quadratic_nonresidue_mod(a, p)
        implies legendre_symbol_neg_one_case(a, p)
} by {
    if p.is_prime and is_unit_quadratic_nonresidue_mod(a, p) {
        unit_quadratic_nonresidue_coprime(a, p)
        a.coprime(p)
        unit_quadratic_nonresidue_is_nonresidue(a, p)
        is_quadratic_nonresidue_mod(a, p)
        legendre_symbol_neg_one_case_of_prime_coprime_nonresidue(p, a)
        legendre_symbol_neg_one_case(a, p)
    }
}

/// The Legendre symbol is zero on the zero residue class.
theorem legendre_symbol_zero_of_congr_zero(a: Nat, p: Nat) {
    a.congr_mod(Nat.0, p) implies legendre_symbol(a, p) = Int.0
} by {
    if a.congr_mod(Nat.0, p) {
        legendre_symbol_zero_case_of_congr_zero(a, p)
        legendre_symbol_zero_case(a, p)
        legendre_symbol(a, p) = Int.0
    }
}

/// The Legendre symbol of zero is zero.
theorem legendre_symbol_zero(p: Nat) {
    legendre_symbol(Nat.0, p) = Int.0
} by {
    Nat.0.congr_mod(Nat.0, p)
    legendre_symbol_zero_of_congr_zero(Nat.0, p)
    legendre_symbol(Nat.0, p) = Int.0
}

/// If neither the zero nor the one case holds, the Legendre symbol is `-1`.
theorem legendre_symbol_neg_one_of_not_zero_not_one_case(a: Nat, p: Nat) {
    not legendre_symbol_zero_case(a, p) and not legendre_symbol_one_case(a, p)
        implies legendre_symbol(a, p) = -Int.1
} by {
        if not legendre_symbol_zero_case(a, p) and not legendre_symbol_one_case(a, p) {
            if legendre_symbol_zero_case(a, p) {
                false
            } else {
                if legendre_symbol_one_case(a, p) {
                    false
                } else {
                    let s: Int = if legendre_symbol_zero_case(a, p) {
                        Int.0
                    } else {
                        if legendre_symbol_one_case(a, p) {
                            Int.1
                        } else {
                            -Int.1
                        }
                    }
                    s = legendre_symbol(a, p)
                    s = -Int.1
                    legendre_symbol(a, p) = -Int.1
                }
            }
        }
}

/// The Legendre symbol is minus one on nonzero quadratic nonresidues.
theorem legendre_symbol_neg_one_of_nonzero_nonresidue(a: Nat, p: Nat) {
    not a.congr_mod(Nat.0, p) and is_quadratic_nonresidue_mod(a, p)
        implies legendre_symbol(a, p) = -Int.1
} by {
    if not a.congr_mod(Nat.0, p) and is_quadratic_nonresidue_mod(a, p) {
        legendre_symbol_neg_one_case_of_nonzero_nonresidue(a, p)
        legendre_symbol_neg_one_case(a, p)
        if legendre_symbol_zero_case(a, p) {
            legendre_symbol_zero_case_congr_zero(a, p)
            a.congr_mod(Nat.0, p)
            false
        }
        if legendre_symbol_one_case(a, p) {
            legendre_symbol_one_case_residue(a, p)
            is_quadratic_residue_mod(a, p)
            quadratic_nonresidue_not_residue(a, p)
            not is_quadratic_residue_mod(a, p)
            false
        }
        not legendre_symbol_zero_case(a, p) and not legendre_symbol_one_case(a, p)
        legendre_symbol_neg_one_of_not_zero_not_one_case(a, p)
        legendre_symbol(a, p) = -Int.1
    }
}

/// The Legendre symbol is minus one on coprime nonresidues modulo a prime.
theorem legendre_symbol_neg_one_of_prime_coprime_nonresidue(p: Nat, a: Nat) {
    p.is_prime and a.coprime(p) and is_quadratic_nonresidue_mod(a, p)
        implies legendre_symbol(a, p) = -Int.1
} by {
    if p.is_prime and a.coprime(p) and is_quadratic_nonresidue_mod(a, p) {
        prime_coprime_imp_nonzero_congr_mod(p, a)
        not a.congr_mod(Nat.0, p)
        legendre_symbol_neg_one_of_nonzero_nonresidue(a, p)
        legendre_symbol(a, p) = -Int.1
    }
}

/// The Legendre symbol is minus one on unit nonresidues modulo a prime.
theorem legendre_symbol_neg_one_of_prime_unit_nonresidue(p: Nat, a: Nat) {
    p.is_prime and is_unit_quadratic_nonresidue_mod(a, p)
        implies legendre_symbol(a, p) = -Int.1
} by {
    if p.is_prime and is_unit_quadratic_nonresidue_mod(a, p) {
        unit_quadratic_nonresidue_coprime(a, p)
        a.coprime(p)
        unit_quadratic_nonresidue_is_nonresidue(a, p)
        is_quadratic_nonresidue_mod(a, p)
        legendre_symbol_neg_one_of_prime_coprime_nonresidue(p, a)
        legendre_symbol(a, p) = -Int.1
    }
}

/// The Legendre symbol takes only the values `0`, `1`, and `-1`.
theorem legendre_symbol_is_value(a: Nat, p: Nat) {
    is_legendre_symbol_value(legendre_symbol(a, p))
} by {
    if legendre_symbol_zero_case(a, p) {
        legendre_symbol_zero_case_congr_zero(a, p)
        a.congr_mod(Nat.0, p)
        legendre_symbol_zero_of_congr_zero(a, p)
        legendre_symbol(a, p) = Int.0
        is_legendre_symbol_value(legendre_symbol(a, p)) =
            (legendre_symbol(a, p) = Int.0 or legendre_symbol(a, p) = Int.1 or
                legendre_symbol(a, p) = -Int.1)
        is_legendre_symbol_value(legendre_symbol(a, p))
    } else {
        if legendre_symbol_one_case(a, p) {
            legendre_symbol_one_case_residue(a, p)
            is_quadratic_residue_mod(a, p)
            legendre_symbol_one_case_nonzero(a, p)
            not a.congr_mod(Nat.0, p)
            legendre_symbol_one_of_nonzero_residue(a, p)
            legendre_symbol(a, p) = Int.1
            is_legendre_symbol_value(legendre_symbol(a, p)) =
                (legendre_symbol(a, p) = Int.0 or legendre_symbol(a, p) = Int.1 or
                    legendre_symbol(a, p) = -Int.1)
            is_legendre_symbol_value(legendre_symbol(a, p))
        } else {
            not legendre_symbol_zero_case(a, p) and not legendre_symbol_one_case(a, p)
            legendre_symbol_neg_one_of_not_zero_not_one_case(a, p)
            legendre_symbol(a, p) = -Int.1
            is_legendre_symbol_value(legendre_symbol(a, p)) =
                (legendre_symbol(a, p) = Int.0 or legendre_symbol(a, p) = Int.1 or
                    legendre_symbol(a, p) = -Int.1)
            is_legendre_symbol_value(legendre_symbol(a, p))
        }
    }
}

/// If the zero case holds, the Legendre symbol has value `0`.
theorem legendre_symbol_zero_of_zero_case(a: Nat, p: Nat) {
    legendre_symbol_zero_case(a, p) implies legendre_symbol(a, p) = Int.0
} by {
    if legendre_symbol_zero_case(a, p) {
        legendre_symbol_zero_case_congr_zero(a, p)
        a.congr_mod(Nat.0, p)
        legendre_symbol_zero_of_congr_zero(a, p)
        legendre_symbol(a, p) = Int.0
    }
}

/// If the Legendre symbol has value `0`, the zero case holds.
theorem legendre_symbol_zero_case_of_value_zero(a: Nat, p: Nat) {
    legendre_symbol(a, p) = Int.0 implies legendre_symbol_zero_case(a, p)
} by {
    if legendre_symbol(a, p) = Int.0 {
        if legendre_symbol_zero_case(a, p) {
            legendre_symbol_zero_case(a, p)
        } else {
            if legendre_symbol_one_case(a, p) {
                legendre_symbol_one_case_residue(a, p)
                is_quadratic_residue_mod(a, p)
                legendre_symbol_one_case_nonzero(a, p)
                not a.congr_mod(Nat.0, p)
                legendre_symbol_one_of_nonzero_residue(a, p)
                legendre_symbol(a, p) = Int.1
                Int.1 = Int.0
                false
            } else {
                not legendre_symbol_zero_case(a, p) and not legendre_symbol_one_case(a, p)
                legendre_symbol_neg_one_of_not_zero_not_one_case(a, p)
                legendre_symbol(a, p) = -Int.1
                -Int.1 = Int.0
                false
            }
        }
    }
}

/// The Legendre symbol has value `0` exactly in the zero case.
theorem legendre_symbol_value_zero_iff_zero_case(a: Nat, p: Nat) {
    (legendre_symbol(a, p) = Int.0) = legendre_symbol_zero_case(a, p)
} by {
    if legendre_symbol(a, p) = Int.0 {
        legendre_symbol_zero_case_of_value_zero(a, p)
        legendre_symbol_zero_case(a, p)
    }
    if legendre_symbol_zero_case(a, p) {
        legendre_symbol_zero_of_zero_case(a, p)
        legendre_symbol(a, p) = Int.0
    }
    ((legendre_symbol(a, p) = Int.0) = legendre_symbol_zero_case(a, p)) = true
}

/// If the one case holds, the Legendre symbol has value `1`.
theorem legendre_symbol_one_of_one_case(a: Nat, p: Nat) {
    legendre_symbol_one_case(a, p) implies legendre_symbol(a, p) = Int.1
} by {
    if legendre_symbol_one_case(a, p) {
        legendre_symbol_one_case_residue(a, p)
        is_quadratic_residue_mod(a, p)
        legendre_symbol_one_case_nonzero(a, p)
        not a.congr_mod(Nat.0, p)
        legendre_symbol_one_of_nonzero_residue(a, p)
        legendre_symbol(a, p) = Int.1
    }
}

/// If the Legendre symbol has value `1`, the one case holds.
theorem legendre_symbol_one_case_of_value_one(a: Nat, p: Nat) {
    legendre_symbol(a, p) = Int.1 implies legendre_symbol_one_case(a, p)
} by {
    if legendre_symbol(a, p) = Int.1 {
        if legendre_symbol_zero_case(a, p) {
            legendre_symbol_zero_of_zero_case(a, p)
            legendre_symbol(a, p) = Int.0
            Int.0 = Int.1
            false
        } else {
            if legendre_symbol_one_case(a, p) {
                legendre_symbol_one_case(a, p)
            } else {
                not legendre_symbol_zero_case(a, p) and not legendre_symbol_one_case(a, p)
                legendre_symbol_neg_one_of_not_zero_not_one_case(a, p)
                legendre_symbol(a, p) = -Int.1
                -Int.1 = Int.1
                false
            }
        }
    }
}

/// The Legendre symbol has value `1` exactly in the one case.
theorem legendre_symbol_value_one_iff_one_case(a: Nat, p: Nat) {
    (legendre_symbol(a, p) = Int.1) = legendre_symbol_one_case(a, p)
} by {
    if legendre_symbol(a, p) = Int.1 {
        legendre_symbol_one_case_of_value_one(a, p)
        legendre_symbol_one_case(a, p)
    }
    if legendre_symbol_one_case(a, p) {
        legendre_symbol_one_of_one_case(a, p)
        legendre_symbol(a, p) = Int.1
    }
    ((legendre_symbol(a, p) = Int.1) = legendre_symbol_one_case(a, p)) = true
}

/// If neither the zero nor one case holds, the `-1` case holds.
theorem legendre_symbol_neg_one_case_of_not_zero_not_one_case(a: Nat, p: Nat) {
    not legendre_symbol_zero_case(a, p) and not legendre_symbol_one_case(a, p)
        implies legendre_symbol_neg_one_case(a, p)
} by {
    if not legendre_symbol_zero_case(a, p) and not legendre_symbol_one_case(a, p) {
        legendre_symbol_zero_case_iff_congr_zero(a, p)
        not a.congr_mod(Nat.0, p)
        if is_quadratic_residue_mod(a, p) {
            legendre_symbol_one_case_of_nonzero_residue(a, p)
            legendre_symbol_one_case(a, p)
            false
        }
        not is_quadratic_residue_mod(a, p)
        quadratic_nonresidue_of_not_residue(a, p)
        is_quadratic_nonresidue_mod(a, p)
        legendre_symbol_neg_one_case_of_nonzero_nonresidue(a, p)
        legendre_symbol_neg_one_case(a, p)
    }
}

/// If the `-1` case holds, the Legendre symbol has value `-1`.
theorem legendre_symbol_neg_one_of_neg_one_case(a: Nat, p: Nat) {
    legendre_symbol_neg_one_case(a, p) implies legendre_symbol(a, p) = -Int.1
} by {
    if legendre_symbol_neg_one_case(a, p) {
        legendre_symbol_neg_one_case_iff_nonzero_nonresidue(a, p)
        not a.congr_mod(Nat.0, p)
        is_quadratic_nonresidue_mod(a, p)
        legendre_symbol_neg_one_of_nonzero_nonresidue(a, p)
        legendre_symbol(a, p) = -Int.1
    }
}

/// A Legendre value of `-1` excludes the zero case.
theorem legendre_symbol_value_neg_one_not_zero_case(a: Nat, p: Nat) {
    legendre_symbol(a, p) = -Int.1 implies not legendre_symbol_zero_case(a, p)
} by {
    if legendre_symbol(a, p) = -Int.1 {
        if legendre_symbol_zero_case(a, p) {
            legendre_symbol_zero_of_zero_case(a, p)
            legendre_symbol(a, p) = Int.0
            Int.0 = -Int.1
            false
        }
        not legendre_symbol_zero_case(a, p)
    }
}

/// A Legendre value of `-1` excludes the one case.
theorem legendre_symbol_value_neg_one_not_one_case(a: Nat, p: Nat) {
    legendre_symbol(a, p) = -Int.1 implies not legendre_symbol_one_case(a, p)
} by {
    if legendre_symbol(a, p) = -Int.1 {
        if legendre_symbol_one_case(a, p) {
            legendre_symbol_one_of_one_case(a, p)
            legendre_symbol(a, p) = Int.1
            Int.1 = -Int.1
            false
        }
        not legendre_symbol_one_case(a, p)
    }
}

/// If the Legendre symbol has value `-1`, the `-1` case holds.
theorem legendre_symbol_neg_one_case_of_value_neg_one(a: Nat, p: Nat) {
    legendre_symbol(a, p) = -Int.1 implies legendre_symbol_neg_one_case(a, p)
} by {
    if legendre_symbol(a, p) = -Int.1 {
        legendre_symbol_value_neg_one_not_zero_case(a, p)
        legendre_symbol_value_neg_one_not_one_case(a, p)
        not legendre_symbol_zero_case(a, p)
        not legendre_symbol_one_case(a, p)
        legendre_symbol_neg_one_case_of_not_zero_not_one_case(a, p)
        legendre_symbol_neg_one_case(a, p)
    }
}

/// The Legendre symbol has value `-1` exactly in the `-1` case.
theorem legendre_symbol_value_neg_one_iff_neg_one_case(a: Nat, p: Nat) {
    (legendre_symbol(a, p) = -Int.1) = legendre_symbol_neg_one_case(a, p)
} by {
    if legendre_symbol(a, p) = -Int.1 {
        legendre_symbol_neg_one_case_of_value_neg_one(a, p)
        legendre_symbol_neg_one_case(a, p)
    }
    if legendre_symbol_neg_one_case(a, p) {
        legendre_symbol_neg_one_of_neg_one_case(a, p)
        legendre_symbol(a, p) = -Int.1
    }
    ((legendre_symbol(a, p) = -Int.1) = legendre_symbol_neg_one_case(a, p)) = true
}

/// Over a prime modulus, the `-1` case is a unit quadratic nonresidue.
theorem legendre_symbol_neg_one_case_unit_nonresidue_prime(p: Nat, a: Nat) {
    p.is_prime and legendre_symbol_neg_one_case(a, p)
        implies is_unit_quadratic_nonresidue_mod(a, p)
} by {
    if p.is_prime and legendre_symbol_neg_one_case(a, p) {
        legendre_symbol_neg_one_case_iff_nonzero_nonresidue(a, p)
        not a.congr_mod(Nat.0, p)
        is_quadratic_nonresidue_mod(a, p)
        prime_nonzero_congr_mod_imp_coprime(p, a)
        a.coprime(p)
        quadratic_nonresidue_coprime_is_unit(a, p)
        is_unit_quadratic_nonresidue_mod(a, p)
    }
}

/// Over a prime modulus, the `-1` case is equivalent to being a unit quadratic
/// nonresidue.
theorem legendre_symbol_neg_one_case_iff_prime_unit_nonresidue(p: Nat, a: Nat) {
    p.is_prime
        implies (legendre_symbol_neg_one_case(a, p) = is_unit_quadratic_nonresidue_mod(a, p))
} by {
    if p.is_prime {
        if legendre_symbol_neg_one_case(a, p) {
            legendre_symbol_neg_one_case_unit_nonresidue_prime(p, a)
            is_unit_quadratic_nonresidue_mod(a, p)
        }
        if is_unit_quadratic_nonresidue_mod(a, p) {
            legendre_symbol_neg_one_case_of_prime_unit_nonresidue(p, a)
            legendre_symbol_neg_one_case(a, p)
        }
        (legendre_symbol_neg_one_case(a, p) = is_unit_quadratic_nonresidue_mod(a, p)) = true
    }
}

/// Over a prime modulus, value `0` is equivalent to not being coprime.
theorem legendre_symbol_value_zero_iff_not_coprime_prime(p: Nat, a: Nat) {
    p.is_prime implies ((legendre_symbol(a, p) = Int.0) = not a.coprime(p))
} by {
    if p.is_prime {
        if legendre_symbol(a, p) = Int.0 {
            legendre_symbol_zero_case_of_value_zero(a, p)
            legendre_symbol_zero_case(a, p)
            legendre_symbol_zero_case_imp_not_coprime_prime(p, a)
            not a.coprime(p)
        }
        if not a.coprime(p) {
            legendre_symbol_zero_case_of_not_coprime_prime(p, a)
            legendre_symbol_zero_case(a, p)
            legendre_symbol_zero_of_zero_case(a, p)
            legendre_symbol(a, p) = Int.0
        }
        ((legendre_symbol(a, p) = Int.0) = not a.coprime(p)) = true
    }
}

/// Over a prime modulus, value `1` is equivalent to being a unit quadratic
/// residue.
theorem legendre_symbol_value_one_iff_prime_unit_quadratic_residue(p: Nat, a: Nat) {
    p.is_prime
        implies ((legendre_symbol(a, p) = Int.1) = is_unit_quadratic_residue_mod(a, p))
} by {
    if p.is_prime {
        legendre_symbol_value_one_iff_one_case(a, p)
        legendre_symbol_one_case_iff_prime_unit_quadratic_residue(p, a)
        (legendre_symbol(a, p) = Int.1) = legendre_symbol_one_case(a, p)
        legendre_symbol_one_case(a, p) = is_unit_quadratic_residue_mod(a, p)
        ((legendre_symbol(a, p) = Int.1) = is_unit_quadratic_residue_mod(a, p)) = true
    }
}

/// A coprime class modulo a prime has Legendre value `1` or `-1`.
theorem legendre_symbol_prime_coprime_unit_value_cases(p: Nat, a: Nat) {
    p.is_prime and a.coprime(p)
        implies legendre_symbol(a, p) = Int.1 or legendre_symbol(a, p) = -Int.1
} by {
    if p.is_prime and a.coprime(p) {
        if is_quadratic_residue_mod(a, p) {
            quadratic_residue_coprime_is_unit(a, p)
            is_unit_quadratic_residue_mod(a, p)
            legendre_symbol_one_of_prime_unit_quadratic_residue(p, a)
            legendre_symbol(a, p) = Int.1
        } else {
            quadratic_nonresidue_of_not_residue(a, p)
            is_quadratic_nonresidue_mod(a, p)
            legendre_symbol_neg_one_of_prime_coprime_nonresidue(p, a)
            legendre_symbol(a, p) = -Int.1
        }
    }
}

/// A coprime class modulo a prime does not have Legendre value `0`.
theorem legendre_symbol_prime_coprime_not_zero(p: Nat, a: Nat) {
    p.is_prime and a.coprime(p) implies legendre_symbol(a, p) != Int.0
} by {
    if p.is_prime and a.coprime(p) {
        if legendre_symbol(a, p) = Int.0 {
            legendre_symbol_value_zero_iff_not_coprime_prime(p, a)
            not a.coprime(p)
            false
        }
    }
}

/// Congruent targets preserve the Legendre `-1` case.
theorem legendre_symbol_neg_one_case_congr_mod(a: Nat, b: Nat, p: Nat) {
    legendre_symbol_neg_one_case(a, p) and a.congr_mod(b, p)
        implies legendre_symbol_neg_one_case(b, p)
} by {
    if legendre_symbol_neg_one_case(a, p) and a.congr_mod(b, p) {
        legendre_symbol_neg_one_case_iff_nonzero_nonresidue(a, p)
        not a.congr_mod(Nat.0, p)
        legendre_symbol_neg_one_case_nonresidue(a, p)
        is_quadratic_nonresidue_mod(a, p)
        if b.congr_mod(Nat.0, p) {
            congr_mod_trans(a, b, Nat.0, p)
            a.congr_mod(Nat.0, p)
            false
        }
        not b.congr_mod(Nat.0, p)
        quadratic_nonresidue_congr_mod(a, b, p)
        is_quadratic_nonresidue_mod(b, p)
        legendre_symbol_neg_one_case_of_nonzero_nonresidue(b, p)
        legendre_symbol_neg_one_case(b, p)
    }
}

/// Congruent targets have equivalent Legendre `-1` cases.
theorem legendre_symbol_neg_one_case_congr_mod_iff(a: Nat, b: Nat, p: Nat) {
    a.congr_mod(b, p) implies
        legendre_symbol_neg_one_case(a, p) = legendre_symbol_neg_one_case(b, p)
} by {
    if a.congr_mod(b, p) {
        if legendre_symbol_neg_one_case(a, p) {
            legendre_symbol_neg_one_case_congr_mod(a, b, p)
            legendre_symbol_neg_one_case(b, p)
        }
        if legendre_symbol_neg_one_case(b, p) {
            congr_mod_symm(a, b, p)
            b.congr_mod(a, p)
            legendre_symbol_neg_one_case_congr_mod(b, a, p)
            legendre_symbol_neg_one_case(a, p)
        }
        (legendre_symbol_neg_one_case(a, p) = legendre_symbol_neg_one_case(b, p)) = true
    }
}

/// Congruent targets have equivalent Legendre value `0` predicates.
theorem legendre_symbol_value_zero_congr_mod_iff(a: Nat, b: Nat, p: Nat) {
    a.congr_mod(b, p) implies
        ((legendre_symbol(a, p) = Int.0) = (legendre_symbol(b, p) = Int.0))
} by {
    if a.congr_mod(b, p) {
        legendre_symbol_value_zero_iff_zero_case(a, p)
        legendre_symbol_value_zero_iff_zero_case(b, p)
        legendre_symbol_zero_case_congr_mod_iff(a, b, p)
        (legendre_symbol(a, p) = Int.0) = legendre_symbol_zero_case(a, p)
        (legendre_symbol(b, p) = Int.0) = legendre_symbol_zero_case(b, p)
        legendre_symbol_zero_case(a, p) = legendre_symbol_zero_case(b, p)
        ((legendre_symbol(a, p) = Int.0) = (legendre_symbol(b, p) = Int.0)) = true
    }
}

/// Congruent targets have equivalent Legendre value `1` predicates.
theorem legendre_symbol_value_one_congr_mod_iff(a: Nat, b: Nat, p: Nat) {
    a.congr_mod(b, p) implies
        ((legendre_symbol(a, p) = Int.1) = (legendre_symbol(b, p) = Int.1))
} by {
    if a.congr_mod(b, p) {
        legendre_symbol_value_one_iff_one_case(a, p)
        legendre_symbol_value_one_iff_one_case(b, p)
        legendre_symbol_one_case_congr_mod_iff(a, b, p)
        (legendre_symbol(a, p) = Int.1) = legendre_symbol_one_case(a, p)
        (legendre_symbol(b, p) = Int.1) = legendre_symbol_one_case(b, p)
        legendre_symbol_one_case(a, p) = legendre_symbol_one_case(b, p)
        ((legendre_symbol(a, p) = Int.1) = (legendre_symbol(b, p) = Int.1)) = true
    }
}

/// Congruent targets have equal Legendre symbols.
theorem legendre_symbol_congr_mod(a: Nat, b: Nat, p: Nat) {
    a.congr_mod(b, p) implies legendre_symbol(a, p) = legendre_symbol(b, p)
} by {
    if a.congr_mod(b, p) {
        legendre_symbol_is_value(a, p)
        is_legendre_symbol_value(legendre_symbol(a, p)) =
            (legendre_symbol(a, p) = Int.0 or
                legendre_symbol(a, p) = Int.1 or
                legendre_symbol(a, p) = -Int.1)
        if legendre_symbol(a, p) = Int.0 {
            legendre_symbol_value_zero_congr_mod_iff(a, b, p)
            legendre_symbol(b, p) = Int.0
            legendre_symbol(a, p) = legendre_symbol(b, p)
        } else {
            if legendre_symbol(a, p) = Int.1 {
                legendre_symbol_value_one_congr_mod_iff(a, b, p)
                legendre_symbol(b, p) = Int.1
                legendre_symbol(a, p) = legendre_symbol(b, p)
            } else {
                legendre_symbol(a, p) = -Int.1
                legendre_symbol_value_neg_one_iff_neg_one_case(a, p)
                legendre_symbol_neg_one_case(a, p)
                legendre_symbol_neg_one_case_congr_mod_iff(a, b, p)
                legendre_symbol_neg_one_case(b, p)
                legendre_symbol_value_neg_one_iff_neg_one_case(b, p)
                legendre_symbol(b, p) = -Int.1
                legendre_symbol(a, p) = legendre_symbol(b, p)
            }
        }
    }
}

/// Reducing the target modulo the modulus preserves its Legendre symbol.
theorem legendre_symbol_mod(a: Nat, p: Nat) {
    legendre_symbol(a.mod(p), p) = legendre_symbol(a, p)
} by {
    mod_congr_mod_self(a, p)
    a.mod(p).congr_mod(a, p)
    legendre_symbol_congr_mod(a.mod(p), a, p)
}
