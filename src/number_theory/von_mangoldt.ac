from nat import Nat, from_nat, from_nat_zero, from_nat_one, from_nat_add, from_nat_mul, one_exp, exp_ne_zero, zero_or_suc, divides_self, divides_lte, lte_antisymm, lte_trans, lt_not_ref, lt_trans, lt_and_lte, mul_cancel_left, add_zero_right, add_one_right, lt_imp_lte_suc, lte_ref, alt_induction, not_lt_zero, pos_of_ne_zero, lte_mul_both, mul_comm, lt_suc, lte_imp_not_lt, only_zero_lte_zero, lt_diff, pow_add, mul_one_right, lt_or_lte, lt_suc_right, mul_to_zero, gcd_of_prime
from real import Real, log_some_of_pos_exists, exp_log_or_zero, log_one, log_mul, exp_injective, exp_nat_mul, from_nat_real_pos_of_ne_zero
from list import List, map, sum, cons_unique_of_tail_unique_not_contains, unique_same_contains_map_sum_eq
from order import not_lt_imp_gte
from number_theory.factorisation import prime_divisor_is_one_or_self, prime_does_not_divide_one, count_prime_factor, count_prime_factor_pow, count_prime_factor_pow_other, divides_iff_count_prime_factor_le, count_prime_factor_ext, divides_imp_count_prime_factor_le, prime_factor_dichotomy
from nat import has_prime_divisor, divides_trans
from number_theory.divisor_sum import divisor_list, divisor_list_contains_implies, divisor_list_contains_of, divisor_list_is_unique
from number_theory.coprime import coprime_divides_of_divides_mul
from data.basic.witness import choose_witness, choose_witness_unique_eq, choose_witness_spec, exists_unique, exists_unique_intro, exists_unique_eq
numerals Nat
numerals Real

/// True if `p` is the base of a prime-power representation of `n`: `p` is prime
/// and `n = p^k` for some `k >= 1`.
define is_prime_power_base_of(n: Nat) -> (Nat -> Bool) {
    function(p: Nat) {
        p.is_prime and exists(k: Nat) {
            Nat.1 <= k and p.pow(k) = n
        }
    }
}

/// True if `n` is a prime power: a positive power of a prime.
define is_prime_power(n: Nat) -> Bool {
    exists(p: Nat) {
        is_prime_power_base_of(n)(p)
    }
}

/// A prime base of `n` when `n` is a prime power, and an arbitrary natural
/// otherwise.
define prime_power_base(n: Nat) -> Nat {
    choose_witness(is_prime_power_base_of(n))
}

/// The von Mangoldt function `Lambda(n)`: `log p` when `n = p^k` is a prime
/// power, and `0` otherwise.
define von_mangoldt(n: Nat) -> Real {
    if is_prime_power(n) {
        (from_nat[Real](prime_power_base(n))).log.get_or_else(Real.0)
    } else {
        Real.0
    }
}

/// A natural number at least one is nonzero.
theorem nat_one_lte_imp_ne_zero(a: Nat) {
    Nat.1 <= a implies a != Nat.0
} by {
    if Nat.1 <= a {
        if a = Nat.0 {
            Nat.1 <= Nat.0
            lt_suc(Nat.0)
            Nat.0 < Nat.1
            lte_imp_not_lt(Nat.1, Nat.0)
            not (Nat.0 < Nat.1)
            false
        }
        a != Nat.0
    }
}

/// A nonzero natural number is the successor of a natural number.
theorem nat_ne_zero_imp_suc(a: Nat) {
    a != Nat.0 implies exists(b: Nat) { a = b.suc }
} by {
    if a != Nat.0 {
        zero_or_suc(a)
        a = Nat.0 or exists(b: Nat) { a = b.suc }
        if a = Nat.0 {
            false
        }
        exists(b: Nat) { a = b.suc }
    }
}

/// A prime dividing a positive power of `p` divides `p` itself.
theorem prime_divides_pow_imp_divides_base(q: Nat, p: Nat, k: Nat) {
    q.is_prime and k != Nat.0 and q.divides(p.pow(k)) implies q.divides(p)
} by {
    define pred(x: Nat) -> Bool {
        q.is_prime and q.divides(p.pow(x)) implies (x = Nat.0 or q.divides(p))
    }
    if q.is_prime and q.divides(p.pow(Nat.0)) {
        Nat.0 = Nat.0 or q.divides(p)
    }
    pred(Nat.0)
    forall(x: Nat) {
        if pred(x) {
            if q.is_prime and q.divides(p.pow(x.suc)) {
                p.pow(x.suc) = p * p.pow(x)
                q.divides(p * p.pow(x))
                gcd_of_prime(q, p)
                q.gcd(p) = Nat.1 or q.divides(p)
                if not q.divides(p) {
                    q.gcd(p) = Nat.1
                    q.coprime(p)
                    coprime_divides_of_divides_mul(q, p, p.pow(x))
                    q.divides(p.pow(x))
                    pred(x) = (q.is_prime and q.divides(p.pow(x)) implies (x = Nat.0 or q.divides(p)))
                    pred(x)
                    x = Nat.0 or q.divides(p)
                    if x = Nat.0 {
                        p.pow(Nat.0) = Nat.1
                        p.pow(x) = Nat.1
                        q.divides(Nat.1)
                        prime_does_not_divide_one(q)
                        not q.divides(Nat.1)
                        false
                    }
                    q.divides(p)
                    not q.divides(p)
                    false
                }
                q.divides(p)
            }
            pred(x.suc)
        }
    }
    pred(Nat.0) and forall(x: Nat) {
        pred(x) implies pred(x.suc)
    }
    Nat.induction(pred)
    pred(k)
    if q.is_prime and k != Nat.0 and q.divides(p.pow(k)) {
        pred(k) = (q.is_prime and q.divides(p.pow(k)) implies (k = Nat.0 or q.divides(p)))
        pred(k)
        k = Nat.0 or q.divides(p)
        if k = Nat.0 {
            k != Nat.0
            false
        }
        q.divides(p)
    }
}

/// The base of a prime-power representation is unique: if `q^j = p^k` for
/// primes `p`, `q` and positive exponents, then `q = p`.
theorem prime_power_representation_unique(p: Nat, k: Nat, q: Nat, j: Nat) {
    p.is_prime and q.is_prime and Nat.1 <= k and Nat.1 <= j and q.pow(j) = p.pow(k)
        implies q = p
} by {
    if p.is_prime and q.is_prime and Nat.1 <= k and Nat.1 <= j and q.pow(j) = p.pow(k) {
        nat_one_lte_imp_ne_zero(j)
        j != Nat.0
        nat_ne_zero_imp_suc(j)
        exists(m: Nat) { j = m.suc }
        let m: Nat satisfy { j = m.suc }
        q.pow(j) = q.pow(m.suc)
        q.pow(m.suc) = q * q.pow(m)
        q.pow(j) = q * q.pow(m)
        q * q.pow(m) = p.pow(k)
        q.divides(p.pow(k))
        nat_one_lte_imp_ne_zero(k)
        k != Nat.0
        prime_divides_pow_imp_divides_base(q, p, k)
        q.divides(p)
        prime_divisor_is_one_or_self(p, q)
        q = Nat.1 or q = p
        if q = Nat.1 {
            Nat.1 < q
            Nat.1 < Nat.1
            lt_not_ref(Nat.1)
            false
        }
        q = p
    }
}

/// A prime `p` is a prime-power base of `p^k` for `k >= 1`.
theorem is_prime_power_base_of_prime_power(p: Nat, k: Nat) {
    p.is_prime and Nat.1 <= k implies is_prime_power_base_of(p.pow(k))(p)
} by {
    if p.is_prime and Nat.1 <= k {
        exists(h: Nat) {
            Nat.1 <= h and p.pow(h) = p.pow(k)
        }
        is_prime_power_base_of(p.pow(k))(p)
    }
}

/// `p^k` is a prime power for `k >= 1`.
theorem is_prime_power_of_prime_power(p: Nat, k: Nat) {
    p.is_prime and Nat.1 <= k implies is_prime_power(p.pow(k))
} by {
    if p.is_prime and Nat.1 <= k {
        is_prime_power_base_of_prime_power(p, k)
        is_prime_power_base_of(p.pow(k))(p)
        is_prime_power(p.pow(k))
    }
}

/// The prime-power base of `p^k` is `p`.
theorem prime_power_base_of_prime_power(p: Nat, k: Nat) {
    p.is_prime and Nat.1 <= k implies prime_power_base(p.pow(k)) = p
} by {
    if p.is_prime and Nat.1 <= k {
        forall(x: Nat, y: Nat) {
            if is_prime_power_base_of(p.pow(k))(x) and is_prime_power_base_of(p.pow(k))(y) {
                is_prime_power_base_of(p.pow(k))(x) =
                    (x.is_prime and exists(j: Nat) {
                        Nat.1 <= j and x.pow(j) = p.pow(k)
                    })
                let j: Nat satisfy { Nat.1 <= j and x.pow(j) = p.pow(k) }
                prime_power_representation_unique(p, k, x, j)
                x = p
                is_prime_power_base_of(p.pow(k))(y) =
                    (y.is_prime and exists(l: Nat) {
                        Nat.1 <= l and y.pow(l) = p.pow(k)
                    })
                let l: Nat satisfy { Nat.1 <= l and y.pow(l) = p.pow(k) }
                prime_power_representation_unique(p, k, y, l)
                y = p
                x = y
            }
        }
        is_prime_power_base_of_prime_power(p, k)
        is_prime_power_base_of(p.pow(k))(p)
        exists_unique_intro(is_prime_power_base_of(p.pow(k)), p)
        exists_unique(is_prime_power_base_of(p.pow(k)))
        choose_witness_unique_eq(is_prime_power_base_of(p.pow(k)), p)
        prime_power_base(p.pow(k)) = p
    }
}

/// The defining value of the von Mangoldt function: `Lambda(p^k) = log p`
/// for a prime `p` and `k >= 1`.
theorem von_mangoldt_prime_power(p: Nat, k: Nat) {
    p.is_prime and Nat.1 <= k implies
        von_mangoldt(p.pow(k)) = (from_nat[Real](p)).log.get_or_else(Real.0)
} by {
    if p.is_prime and Nat.1 <= k {
        is_prime_power_of_prime_power(p, k)
        is_prime_power(p.pow(k))
        von_mangoldt(p.pow(k)) =
            (from_nat[Real](prime_power_base(p.pow(k)))).log.get_or_else(Real.0)
        prime_power_base_of_prime_power(p, k)
        prime_power_base(p.pow(k)) = p
        von_mangoldt(p.pow(k)) = (from_nat[Real](p)).log.get_or_else(Real.0)
    }
}

/// The value `Lambda(p^k)` is the logarithm of `p` in the sense of the
/// option-valued logarithm.
theorem von_mangoldt_prime_power_log(p: Nat, k: Nat) {
    p.is_prime and Nat.1 <= k implies
        (from_nat[Real](p)).log = Option.some(von_mangoldt(p.pow(k)))
} by {
    if p.is_prime and Nat.1 <= k {
        Nat.1 < p
        p != Nat.0
        from_nat_real_pos_of_ne_zero(p)
        from_nat[Real](p) > Real.0
        log_some_of_pos_exists(from_nat[Real](p))
        let lp: Real satisfy {
            (from_nat[Real](p)).log = Option.some(lp)
        }
        option_get_or_else_some[Real](lp, Real.0)
        option_get_or_else(Option.some(lp), Real.0) = lp
        (from_nat[Real](p)).log.get_or_else(Real.0) = lp
        (from_nat[Real](p)).log = Option.some((from_nat[Real](p)).log.get_or_else(Real.0))
        von_mangoldt_prime_power(p, k)
        von_mangoldt(p.pow(k)) = (from_nat[Real](p)).log.get_or_else(Real.0)
        (from_nat[Real](p)).log = Option.some(von_mangoldt(p.pow(k)))
    }
}

/// A nonzero base divides every positive power of itself.
theorem nat_pow_divides_self(a: Nat, n: Nat) {
    a != Nat.0 and n != Nat.0 implies a.divides(a.pow(n))
} by {
    if a != Nat.0 and n != Nat.0 {
        nat_ne_zero_imp_suc(n)
        exists(m: Nat) { n = m.suc }
        let m: Nat satisfy { n = m.suc }
        a.pow(n) = a.pow(m.suc)
        a.pow(m.suc) = a * a.pow(m)
        a.pow(n) = a * a.pow(m)
        a.divides(a.pow(n))
    }
}

/// If a power of a prime is a prime, the bases are equal.
theorem prime_pow_eq_prime_imp_eq(p: Nat, m: Nat, q: Nat) {
    p.is_prime and q.is_prime and p.pow(m) = q implies p = q
} by {
    if p.is_prime and q.is_prime and p.pow(m) = q {
        if m = Nat.0 {
            p.pow(Nat.0) = Nat.1
            p.pow(m) = Nat.1
            p.pow(m) = q
            q = Nat.1
            Nat.1 < q
            Nat.1 < Nat.1
            lt_not_ref(Nat.1)
            false
        }
        m != Nat.0
        Nat.1 < p
        p != Nat.0
        nat_pow_divides_self(p, m)
        p.divides(p.pow(m))
        p.pow(m) = q
        p.divides(q)
        prime_divisor_is_one_or_self(q, p)
        p = Nat.1 or p = q
        if p = Nat.1 {
            Nat.1 < p
            Nat.1 < Nat.1
            lt_not_ref(Nat.1)
            false
        }
        p = q
    }
}

/// One is not a prime power.
theorem not_is_prime_power_one {
    not is_prime_power(Nat.1)
} by {
    if is_prime_power(Nat.1) {
        is_prime_power(Nat.1) = exists(p: Nat) { is_prime_power_base_of(Nat.1)(p) }
        let p: Nat satisfy { is_prime_power_base_of(Nat.1)(p) }
        is_prime_power_base_of(Nat.1)(p) =
            (p.is_prime and exists(k: Nat) { Nat.1 <= k and p.pow(k) = Nat.1 })
        p.is_prime
        let k: Nat satisfy { Nat.1 <= k and p.pow(k) = Nat.1 }
        p.pow(k) = Nat.1
        nat_one_lte_imp_ne_zero(k)
        k != Nat.0
        nat_ne_zero_imp_suc(k)
        let m: Nat satisfy { k = m.suc }
        p.pow(k) = p.pow(m.suc)
        p.pow(m.suc) = p * p.pow(m)
        p.pow(k) = p * p.pow(m)
        p * p.pow(m) = Nat.1
        p.divides(Nat.1)
        prime_does_not_divide_one(p)
        not p.divides(Nat.1)
        false
    }
}

/// `Lambda(1) = 0`, since one is not a prime power.
theorem von_mangoldt_one {
    von_mangoldt(Nat.1) = Real.0
} by {
    not_is_prime_power_one
    not is_prime_power(Nat.1)
    von_mangoldt(Nat.1) = if is_prime_power(Nat.1) {
        (from_nat[Real](prime_power_base(Nat.1))).log.get_or_else(Real.0)
    } else {
        Real.0
    }
    von_mangoldt(Nat.1) = Real.0
}

/// A product of two distinct primes is not a prime power.
theorem not_is_prime_power_prime_mul(p: Nat, q: Nat) {
    p.is_prime and q.is_prime and p != q implies not is_prime_power(p * q)
} by {
    if p.is_prime and q.is_prime and p != q {
        if is_prime_power(p * q) {
            is_prime_power(p * q) = exists(r: Nat) { is_prime_power_base_of(p * q)(r) }
            let r: Nat satisfy { is_prime_power_base_of(p * q)(r) }
            is_prime_power_base_of(p * q)(r) =
                (r.is_prime and exists(k: Nat) { Nat.1 <= k and r.pow(k) = p * q })
            r.is_prime
            let k: Nat satisfy { Nat.1 <= k and r.pow(k) = p * q }
            r.pow(k) = p * q
            nat_one_lte_imp_ne_zero(k)
            k != Nat.0
            nat_ne_zero_imp_suc(k)
            let m: Nat satisfy { k = m.suc }
            r.pow(k) = r.pow(m.suc)
            r.pow(m.suc) = r * r.pow(m)
            r.pow(k) = r * r.pow(m)
            r * r.pow(m) = p * q
            r.divides(p * q)
            gcd_of_prime(r, p)
            r.gcd(p) = Nat.1 or r.divides(p)
            if r.divides(p) {
                prime_divisor_is_one_or_self(p, r)
                r = Nat.1 or r = p
                if r = Nat.1 {
                    Nat.1 < r
                    Nat.1 < Nat.1
                    lt_not_ref(Nat.1)
                    false
                }
                r = p
                r.pow(k) = p.pow(k)
                p.pow(k) = p * q
                p.pow(k) = p.pow(m.suc)
                p.pow(m.suc) = p * p.pow(m)
                p.pow(k) = p * p.pow(m)
                p * p.pow(m) = p * q
                Nat.1 < p
                p != Nat.0
                mul_cancel_left(p, p.pow(m), q)
                p.pow(m) = q
                prime_pow_eq_prime_imp_eq(p, m, q)
                p = q
                p != q
                false
            } else {
                r.gcd(p) = Nat.1
                r.coprime(p)
                coprime_divides_of_divides_mul(r, p, q)
                r.divides(q)
                prime_divisor_is_one_or_self(q, r)
                r = Nat.1 or r = q
                if r = Nat.1 {
                    Nat.1 < r
                    Nat.1 < Nat.1
                    lt_not_ref(Nat.1)
                    false
                }
                r = q
                r.pow(k) = q.pow(k)
                q.pow(k) = p * q
                q.pow(k) = q.pow(m.suc)
                q.pow(m.suc) = q * q.pow(m)
                q.pow(k) = q * q.pow(m)
                q * q.pow(m) = p * q
                Nat.1 < q
                q != Nat.0
                mul_cancel_left(q, q.pow(m), p)
                q.pow(m) = p
                prime_pow_eq_prime_imp_eq(q, m, p)
                q = p
                p != q
                false
            }
            false
        }
        not is_prime_power(p * q)
    }
}

/// `Lambda(p * q) = 0` for distinct primes `p` and `q`, since a product of
/// two distinct primes is not a prime power.
theorem von_mangoldt_prime_mul(p: Nat, q: Nat) {
    p.is_prime and q.is_prime and p != q implies von_mangoldt(p * q) = Real.0
} by {
    if p.is_prime and q.is_prime and p != q {
        not_is_prime_power_prime_mul(p, q)
        not is_prime_power(p * q)
        von_mangoldt(p * q) = if is_prime_power(p * q) {
            (from_nat[Real](prime_power_base(p * q))).log.get_or_else(Real.0)
        } else {
            Real.0
        }
        von_mangoldt(p * q) = Real.0
    }
}

/// The descending list of powers `p^k, p^(k-1), ..., p, 1`.
define pow_list(p: Nat, k: Nat) -> List[Nat] {
    match k {
        Nat.zero {
            List.cons(Nat.1, List.nil[Nat])
        }
        Nat.suc(j) {
            List.cons(p.pow(k), pow_list(p, j))
        }
    }
}

/// Powers of a nonzero base are at least one.
theorem nat_pow_gte_one(a: Nat, n: Nat) {
    a != Nat.0 implies Nat.1 <= a.pow(n)
} by {
    define pred(x: Nat) -> Bool {
        a != Nat.0 implies Nat.1 <= a.pow(x)
    }
    pred(Nat.0) = (a != Nat.0 implies Nat.1 <= a.pow(Nat.0))
    if a != Nat.0 {
        a.pow(Nat.0) = Nat.1
        Nat.1 <= a.pow(Nat.0)
    }
    pred(Nat.0)
    forall(x: Nat) {
        if pred(x) {
            pred(x) = (a != Nat.0 implies Nat.1 <= a.pow(x))
            if a != Nat.0 {
                a.pow(x.suc) = a * a.pow(x)
                Nat.1 <= a.pow(x)
                pos_of_ne_zero(a)
                Nat.0 < a
                lt_imp_lte_suc(Nat.0, a)
                Nat.1 <= a
                lte_mul_both(a, Nat.1, a.pow(x))
                a * Nat.1 <= a * a.pow(x)
                mul_one_right(a)
                a * Nat.1 = a
                a <= a * a.pow(x)
                lte_trans(Nat.1, a, a * a.pow(x))
                Nat.1 <= a * a.pow(x)
                Nat.1 <= a.pow(x.suc)
            }
            pred(x.suc) = (a != Nat.0 implies Nat.1 <= a.pow(x.suc))
            pred(x.suc)
        }
    }
    pred(Nat.0) and forall(x: Nat) {
        pred(x) implies pred(x.suc)
    }
    Nat.induction(pred)
    pred(n)
}

/// Positive powers of a base greater than one are greater than one.
theorem nat_pow_gt_one(a: Nat, n: Nat) {
    Nat.1 < a and n != Nat.0 implies Nat.1 < a.pow(n)
} by {
    if Nat.1 < a and n != Nat.0 {
        nat_ne_zero_imp_suc(n)
        exists(m: Nat) { n = m.suc }
        let m: Nat satisfy { n = m.suc }
        a.pow(n) = a.pow(m.suc)
        a.pow(m.suc) = a * a.pow(m)
        a.pow(n) = a * a.pow(m)
        a != Nat.0
        nat_pow_gte_one(a, m)
        Nat.1 <= a.pow(m)
        lte_mul_both(a, Nat.1, a.pow(m))
        a * Nat.1 <= a * a.pow(m)
        a * Nat.1 = a
        a <= a * a.pow(m)
        lt_and_lte(Nat.1, a, a * a.pow(m))
        Nat.1 < a * a.pow(m)
        Nat.1 < a.pow(n)
    }
}

/// A power of a base greater than one equals one only at exponent zero.
theorem nat_pow_eq_one_imp_zero(a: Nat, d: Nat) {
    Nat.1 < a and a.pow(d) = Nat.1 implies d = Nat.0
} by {
    if Nat.1 < a and a.pow(d) = Nat.1 {
        if d != Nat.0 {
            nat_pow_gt_one(a, d)
            Nat.1 < a.pow(d)
            a.pow(d) = Nat.1
            Nat.1 < Nat.1
            lt_not_ref(Nat.1)
            false
        }
        d = Nat.0
    }
}

/// Equal powers of a base greater than one have equal exponents.
theorem nat_pow_eq_imp_eq_exp(a: Nat, e: Nat, f: Nat) {
    Nat.1 < a and a.pow(e) = a.pow(f) implies e = f
} by {
    if Nat.1 < a and a.pow(e) = a.pow(f) {
        if e < f {
            lt_diff(e, f)
            let c: Nat satisfy { e + c = f and c != Nat.0 }
            pow_add[Nat](a, e, c)
            a.pow(e) * a.pow(c) = a.pow(e + c)
            a.pow(e + c) = a.pow(f)
            a.pow(e) * a.pow(c) = a.pow(f)
            a.pow(f) = a.pow(e)
            a.pow(e) * a.pow(c) = a.pow(e)
            a != Nat.0
            exp_ne_zero(a, e)
            a.pow(e) != Nat.0
            mul_cancel_left(a.pow(e), a.pow(c), Nat.1)
            a.pow(c) = Nat.1
            nat_pow_gt_one(a, c)
            Nat.1 < a.pow(c)
            Nat.1 < Nat.1
            lt_not_ref(Nat.1)
            false
        }
        if f < e {
            lt_diff(f, e)
            let d: Nat satisfy { f + d = e and d != Nat.0 }
            pow_add[Nat](a, f, d)
            a.pow(f) * a.pow(d) = a.pow(f + d)
            a.pow(f + d) = a.pow(e)
            a.pow(f) * a.pow(d) = a.pow(e)
            a.pow(e) = a.pow(f)
            a.pow(f) * a.pow(d) = a.pow(f)
            a != Nat.0
            exp_ne_zero(a, f)
            a.pow(f) != Nat.0
            mul_cancel_left(a.pow(f), a.pow(d), Nat.1)
            a.pow(d) = Nat.1
            nat_pow_gt_one(a, d)
            Nat.1 < a.pow(d)
            Nat.1 < Nat.1
            lt_not_ref(Nat.1)
            false
        }
        not e < f
        not f < e
        not_lt_imp_gte(f, e)
        f >= e
        not_lt_imp_gte(e, f)
        e >= f
        lte_antisymm(e, f)
        e = f
    }
}

/// A base divides every power whose exponent it bounds: `e <= k` gives `p^e | p^k`.
theorem nat_pow_divides_pow(a: Nat, e: Nat, k: Nat) {
    a != Nat.0 and e <= k implies a.pow(e).divides(a.pow(k))
} by {
    if a != Nat.0 and e <= k {
        if e < k {
            lt_diff(e, k)
            let c: Nat satisfy { e + c = k and c != Nat.0 }
            pow_add[Nat](a, e, c)
            a.pow(e) * a.pow(c) = a.pow(e + c)
            a.pow(e + c) = a.pow(k)
            a.pow(e) * a.pow(c) = a.pow(k)
            a.pow(e).divides(a.pow(k))
        } else {
            e = k
            divides_self(a.pow(e))
            a.pow(e).divides(a.pow(e))
            a.pow(e).divides(a.pow(k))
        }
        a.pow(e).divides(a.pow(k))
    }
}

/// Natural powers commute with the embedding into the reals.
theorem from_nat_real_pow(p: Nat, k: Nat) {
    from_nat[Real](p.pow(k)) = from_nat[Real](p).pow(k)
} by {
    define pred(x: Nat) -> Bool {
        from_nat[Real](p.pow(x)) = from_nat[Real](p).pow(x)
    }
    p.pow(Nat.0) = Nat.1
    from_nat[Real](Nat.1) = Real.1
    from_nat[Real](p.pow(Nat.0)) = Real.1
    from_nat[Real](p).pow(Nat.0) = Real.1
    pred(Nat.0)
    forall(x: Nat) {
        if pred(x) {
            p.pow(x.suc) = p * p.pow(x)
            from_nat_mul[Real](p, p.pow(x))
            from_nat[Real](p * p.pow(x)) = from_nat[Real](p) * from_nat[Real](p.pow(x))
            from_nat[Real](p.pow(x.suc)) = from_nat[Real](p) * from_nat[Real](p.pow(x))
            pred(x)
            from_nat[Real](p.pow(x)) = from_nat[Real](p).pow(x)
            from_nat[Real](p.pow(x.suc)) = from_nat[Real](p) * from_nat[Real](p).pow(x)
            from_nat[Real](p) * from_nat[Real](p).pow(x) = from_nat[Real](p).pow(x.suc)
            pred(x.suc)
        }
    }
    pred(Nat.0) and forall(x: Nat) {
        pred(x) implies pred(x.suc)
    }
    Nat.induction(pred)
    pred(k)
}

/// A divisor of a prime power has a bounded prime count at that prime.
theorem divisor_of_prime_power_count_bounded(p: Nat, k: Nat, d: Nat) {
    p.is_prime and Nat.0 < d and d.divides(p.pow(k)) implies
        count_prime_factor(p, d) <= k
} by {
    if p.is_prime and Nat.0 < d and d.divides(p.pow(k)) {
        d != Nat.0
        Nat.1 < p
        p != Nat.0
        exp_ne_zero(p, k)
        p.pow(k) != Nat.0
        divides_imp_count_prime_factor_le(p, d, p.pow(k))
        count_prime_factor(p, d) <= count_prime_factor(p, p.pow(k))
        count_prime_factor_pow(p, k)
        count_prime_factor(p, p.pow(k)) = k
        count_prime_factor(p, d) <= k
    }
}

/// A divisor of a prime power is a power of that prime, with exponent the
/// prime count.
theorem divisor_of_prime_power_eq_pow(p: Nat, k: Nat, d: Nat) {
    p.is_prime and Nat.0 < d and d.divides(p.pow(k)) implies
        d = p.pow(count_prime_factor(p, d))
} by {
    if p.is_prime and Nat.0 < d and d.divides(p.pow(k)) {
        d != Nat.0
        Nat.1 < p
        p != Nat.0
        exp_ne_zero(p, k)
        p.pow(k) != Nat.0
        exp_ne_zero(p, count_prime_factor(p, d))
        p.pow(count_prime_factor(p, d)) != Nat.0
        forall(q: Nat) {
            if q.is_prime {
                if q = p {
                    count_prime_factor(q, d) = count_prime_factor(p, d)
                    count_prime_factor_pow(p, count_prime_factor(p, d))
                    count_prime_factor(p, p.pow(count_prime_factor(p, d))) =
                        count_prime_factor(p, d)
                    count_prime_factor(q, p.pow(count_prime_factor(p, d))) =
                        count_prime_factor(q, d)
                } else {
                    q != p
                    divides_imp_count_prime_factor_le(q, d, p.pow(k))
                    count_prime_factor(q, d) <= count_prime_factor(q, p.pow(k))
                    count_prime_factor_pow_other(p, q, k)
                    count_prime_factor(q, p.pow(k)) = Nat.0
                    count_prime_factor(q, d) <= Nat.0
                    only_zero_lte_zero(count_prime_factor(q, d))
                    count_prime_factor(q, d) = Nat.0
                    count_prime_factor_pow_other(p, q, count_prime_factor(p, d))
                    count_prime_factor(q, p.pow(count_prime_factor(p, d))) = Nat.0
                    count_prime_factor(q, d) =
                        count_prime_factor(q, p.pow(count_prime_factor(p, d)))
                }
                count_prime_factor(q, d) =
                    count_prime_factor(q, p.pow(count_prime_factor(p, d)))
            }
        }
        count_prime_factor_ext(d, p.pow(count_prime_factor(p, d)))
        d = p.pow(count_prime_factor(p, d))
    }
}

/// A divisor of a prime power is a power of that prime: `d | p^k` with
/// `d > 0` gives `d = p^e` for some `e <= k`.
theorem divisor_of_prime_power_is_power(p: Nat, k: Nat, d: Nat) {
    p.is_prime and Nat.0 < d and d.divides(p.pow(k)) implies
        exists(e: Nat) { e <= k and d = p.pow(e) }
} by {
    if p.is_prime and Nat.0 < d and d.divides(p.pow(k)) {
        divisor_of_prime_power_count_bounded(p, k, d)
        count_prime_factor(p, d) <= k
        divisor_of_prime_power_eq_pow(p, k, d)
        d = p.pow(count_prime_factor(p, d))
        exists(e: Nat) { e <= k and d = p.pow(e) }
    }
}


/// A bound persists under the successor.
theorem nat_lte_imp_lte_suc(a: Nat, b: Nat) {
    a <= b implies a <= b.suc
} by {
    if a <= b {
        lt_suc(b)
        b < b.suc
        lt_imp_lte_suc(b, b.suc)
        b <= b.suc
        lte_trans(a, b, b.suc)
        a <= b.suc
    }
}

/// A bound at a successor that is not the successor itself persists.
theorem nat_lte_suc_imp_lte_of_ne(a: Nat, b: Nat) {
    a <= b.suc and a != b.suc implies a <= b
} by {
    if a <= b.suc and a != b.suc {
        lt_or_lte(a, b.suc)
        a < b.suc or b.suc <= a
        if b.suc <= a {
            lte_antisymm(a, b.suc)
            a = b.suc
            a != b.suc
            false
        }
        a < b.suc
        lt_suc_right(a, b)
        a = b or a < b
        if a < b {
            lt_imp_lte_suc(a, b)
            a <= b
        } else {
            a = b
            a <= b
        }
        a <= b
    }
}

/// Membership in the descending power list means being a power of the base.
theorem pow_list_contains_imp_power(p: Nat, k: Nat, d: Nat) {
    Nat.1 < p and pow_list(p, k).contains(d) implies
        exists(e: Nat) { e <= k and d = p.pow(e) }
} by {
    define pred(x: Nat) -> Bool {
        pow_list(p, x).contains(d) implies
            exists(e: Nat) { e <= x and d = p.pow(e) }
    }
    pow_list(p, Nat.0) = List.cons(Nat.1, List.nil[Nat])
    if pow_list(p, Nat.0).contains(d) {
        List.cons(Nat.1, List.nil[Nat]).contains(d)
        d = Nat.1 or List.nil[Nat].contains(d)
        not List.nil[Nat].contains(d)
        d = Nat.1
        p.pow(Nat.0) = Nat.1
        d = p.pow(Nat.0)
        exists(e: Nat) { e <= Nat.0 and d = p.pow(e) }
    }
    pred(Nat.0)
    forall(x: Nat) {
        if pred(x) {
            if pow_list(p, x.suc).contains(d) {
                pow_list(p, x.suc) = List.cons(p.pow(x.suc), pow_list(p, x))
                List.cons(p.pow(x.suc), pow_list(p, x)).contains(d)
                if d = p.pow(x.suc) {
                    exists(e: Nat) { e <= x.suc and d = p.pow(e) }
                } else {
                    d != p.pow(x.suc)
                    pow_list(p, x).contains(d)
                    pred(x)
                    exists(e: Nat) { e <= x and d = p.pow(e) }
                    let e: Nat satisfy { e <= x and d = p.pow(e) }
                    nat_lte_imp_lte_suc(e, x)
                    e <= x.suc
                    exists(witness: Nat) {
                        witness = e and witness <= x.suc and d = p.pow(witness)
                    }
                }
                exists(e: Nat) { e <= x.suc and d = p.pow(e) }
            }
            pred(x.suc)
        }
    }
    pred(Nat.0) and forall(x: Nat) {
        pred(x) implies pred(x.suc)
    }
    Nat.induction(pred)
    pred(k)
    if Nat.1 < p and pow_list(p, k).contains(d) {
        pred(k)
        exists(e: Nat) { e <= k and d = p.pow(e) }
    }
}

/// A power of the base with bounded exponent lies in the descending power list.
theorem power_imp_pow_list_contains(p: Nat, k: Nat, d: Nat) {
    Nat.1 < p and exists(e: Nat) { e <= k and d = p.pow(e) } implies
        pow_list(p, k).contains(d)
} by {
    define pred(x: Nat) -> Bool {
        exists(e: Nat) { e <= x and d = p.pow(e) } implies pow_list(p, x).contains(d)
    }
    if exists(e: Nat) { e <= Nat.0 and d = p.pow(e) } {
        let e: Nat satisfy { e <= Nat.0 and d = p.pow(e) }
        only_zero_lte_zero(e)
        e = Nat.0
        p.pow(Nat.0) = Nat.1
        d = Nat.1
        pow_list(p, Nat.0) = List.cons(Nat.1, List.nil[Nat])
        List.cons(Nat.1, List.nil[Nat]).contains(Nat.1)
        pow_list(p, Nat.0).contains(d)
    }
    pred(Nat.0)
    forall(x: Nat) {
        if pred(x) {
            if exists(e: Nat) { e <= x.suc and d = p.pow(e) } {
                let e: Nat satisfy { e <= x.suc and d = p.pow(e) }
                if e = x.suc {
                    d = p.pow(x.suc)
                    pow_list(p, x.suc) = List.cons(p.pow(x.suc), pow_list(p, x))
                    List.cons(p.pow(x.suc), pow_list(p, x)).contains(p.pow(x.suc))
                    pow_list(p, x.suc).contains(d)
                } else {
                    e != x.suc
                    nat_lte_suc_imp_lte_of_ne(e, x)
                    e <= x
                    pred(x)
                    pow_list(p, x).contains(d)
                    pow_list(p, x.suc) = List.cons(p.pow(x.suc), pow_list(p, x))
                    List.cons(p.pow(x.suc), pow_list(p, x)).contains(d)
                    pow_list(p, x.suc).contains(d)
                }
                pow_list(p, x.suc).contains(d)
            }
            pred(x.suc)
        }
    }
    pred(Nat.0) and forall(x: Nat) {
        pred(x) implies pred(x.suc)
    }
    Nat.induction(pred)
    pred(k)
    if Nat.1 < p and exists(e: Nat) { e <= k and d = p.pow(e) } {
        pred(k)
        pow_list(p, k).contains(d)
    }
}

/// The descending power list of a base greater than one has no repeats.
theorem pow_list_unique(p: Nat, k: Nat) {
    Nat.1 < p implies pow_list(p, k).is_unique
} by {
    define pred(x: Nat) -> Bool {
        pow_list(p, x).is_unique
    }
    pow_list(p, Nat.0) = List.cons(Nat.1, List.nil[Nat])
    List.cons(Nat.1, List.nil[Nat]).is_unique
    pred(Nat.0)
    forall(x: Nat) {
        if pred(x) {
            if pow_list(p, x).contains(p.pow(x.suc)) {
                pow_list_contains_imp_power(p, x, p.pow(x.suc))
                exists(e: Nat) { e <= x and p.pow(x.suc) = p.pow(e) }
                let e: Nat satisfy { e <= x and p.pow(x.suc) = p.pow(e) }
                nat_pow_eq_imp_eq_exp(p, x.suc, e)
                x.suc = e
                e <= x
                x.suc <= x
                lt_suc(x)
                x < x.suc
                lte_imp_not_lt(x.suc, x)
                not (x < x.suc)
                false
            }
            not pow_list(p, x).contains(p.pow(x.suc))
            pow_list(p, x.suc) = List.cons(p.pow(x.suc), pow_list(p, x))
            cons_unique_of_tail_unique_not_contains(p.pow(x.suc), pow_list(p, x))
            List.cons(p.pow(x.suc), pow_list(p, x)).is_unique
            pred(x.suc)
        }
    }
    pred(Nat.0) and forall(x: Nat) {
        pred(x) implies pred(x.suc)
    }
    Nat.induction(pred)
    pred(k)
    if Nat.1 < p {
        pred(k)
    }
}

/// A member of the descending power list divides `p^k`.
theorem pow_list_contains_imp_divisor_list(p: Nat, k: Nat, d: Nat) {
    p.is_prime and pow_list(p, k).contains(d) implies divisor_list(p.pow(k)).contains(d)
} by {
    if p.is_prime and pow_list(p, k).contains(d) {
        Nat.1 < p
        p != Nat.0
        pow_list_contains_imp_power(p, k, d)
        exists(e: Nat) { e <= k and d = p.pow(e) }
        let e: Nat satisfy { e <= k and d = p.pow(e) }
        nat_pow_divides_pow(p, e, k)
        p.pow(e).divides(p.pow(k))
        d.divides(p.pow(k))
        nat_pow_gte_one(p, e)
        Nat.1 <= p.pow(e)
        Nat.0 < p.pow(e)
        Nat.0 < d
        exp_ne_zero(p, k)
        p.pow(k) != Nat.0
        pos_of_ne_zero(p.pow(k))
        Nat.0 < p.pow(k)
        divisor_list_contains_of(p.pow(k), d)
        divisor_list(p.pow(k)).contains(d)
    }
}

/// A divisor of `p^k` is a member of the descending power list.
theorem divisor_list_pow_contains_imp_pow_list(p: Nat, k: Nat, d: Nat) {
    p.is_prime and divisor_list(p.pow(k)).contains(d) implies pow_list(p, k).contains(d)
} by {
    if p.is_prime and divisor_list(p.pow(k)).contains(d) {
        Nat.1 < p
        divisor_list_contains_implies(p.pow(k), d)
        Nat.0 < d and d.divides(p.pow(k))
        divisor_of_prime_power_is_power(p, k, d)
        exists(e: Nat) { e <= k and d = p.pow(e) }
        power_imp_pow_list_contains(p, k, d)
        pow_list(p, k).contains(d)
    }
}

/// The divisor list of `p^k` and the descending power list carry the same
/// members, so the divisor sums of `Lambda` agree.
theorem divisor_list_prime_power_sum_eq_pow_list_sum(p: Nat, k: Nat) {
    p.is_prime implies
        sum(map(divisor_list(p.pow(k)), von_mangoldt)) =
            sum(map(pow_list(p, k), von_mangoldt))
} by {
    if p.is_prime {
        Nat.1 < p
        divisor_list_is_unique(p.pow(k))
        divisor_list(p.pow(k)).is_unique
        pow_list_unique(p, k)
        pow_list(p, k).is_unique
        forall(d: Nat) {
            if divisor_list(p.pow(k)).contains(d) {
                divisor_list_pow_contains_imp_pow_list(p, k, d)
                pow_list(p, k).contains(d)
                divisor_list(p.pow(k)).contains(d) = pow_list(p, k).contains(d)
            }
            if not divisor_list(p.pow(k)).contains(d) {
                if pow_list(p, k).contains(d) {
                    pow_list_contains_imp_divisor_list(p, k, d)
                    divisor_list(p.pow(k)).contains(d)
                    not divisor_list(p.pow(k)).contains(d)
                    false
                }
                not pow_list(p, k).contains(d)
                divisor_list(p.pow(k)).contains(d) = pow_list(p, k).contains(d)
            }
            divisor_list(p.pow(k)).contains(d) = pow_list(p, k).contains(d)
        }
        unique_same_contains_map_sum_eq(
            divisor_list(p.pow(k)), pow_list(p, k), von_mangoldt)
        sum(map(divisor_list(p.pow(k)), von_mangoldt)) =
            sum(map(pow_list(p, k), von_mangoldt))
    }
}

/// Base case of the power-list sum: summing `Lambda` over `[1]` gives zero.
theorem pow_list_sum_von_mangoldt_zero(p: Nat) {
    p.is_prime implies
        sum(map(pow_list(p, Nat.0), von_mangoldt)) =
            from_nat[Real](Nat.0) * (from_nat[Real](p)).log.get_or_else(Real.0)
} by {
    if p.is_prime {
        pow_list(p, Nat.0) = List.cons(Nat.1, List.nil[Nat])
        map(List.cons(Nat.1, List.nil[Nat]), von_mangoldt) =
            List.cons(von_mangoldt(Nat.1), map(List.nil[Nat], von_mangoldt))
        map(List.nil[Nat], von_mangoldt) = List.nil[Real]
        map(List.cons(Nat.1, List.nil[Nat]), von_mangoldt) =
            List.cons(von_mangoldt(Nat.1), List.nil[Real])
        sum(List.cons(von_mangoldt(Nat.1), List.nil[Real])) =
            von_mangoldt(Nat.1) + sum(List.nil[Real])
        von_mangoldt_one
        von_mangoldt(Nat.1) = Real.0
        sum(List.nil[Real]) = Real.0
        sum(map(pow_list(p, Nat.0), von_mangoldt)) = Real.0
        from_nat[Real](Nat.0) = Real.0
        Real.0 * (from_nat[Real](p)).log.get_or_else(Real.0) = Real.0
        sum(map(pow_list(p, Nat.0), von_mangoldt)) =
            from_nat[Real](Nat.0) * (from_nat[Real](p)).log.get_or_else(Real.0)
    }
}

/// Step case of the power-list sum: extending the exponent adds one `log p`.
theorem pow_list_sum_von_mangoldt_suc(p: Nat, x: Nat) {
    p.is_prime and
        sum(map(pow_list(p, x), von_mangoldt)) =
            from_nat[Real](x) * (from_nat[Real](p)).log.get_or_else(Real.0)
        implies
            sum(map(pow_list(p, x.suc), von_mangoldt)) =
                from_nat[Real](x.suc) * (from_nat[Real](p)).log.get_or_else(Real.0)
} by {
    if p.is_prime and
            sum(map(pow_list(p, x), von_mangoldt)) =
                from_nat[Real](x) * (from_nat[Real](p)).log.get_or_else(Real.0) {
        pow_list(p, x.suc) = List.cons(p.pow(x.suc), pow_list(p, x))
        map(List.cons(p.pow(x.suc), pow_list(p, x)), von_mangoldt) =
            List.cons(von_mangoldt(p.pow(x.suc)), map(pow_list(p, x), von_mangoldt))
        sum(List.cons(von_mangoldt(p.pow(x.suc)), map(pow_list(p, x), von_mangoldt))) =
            von_mangoldt(p.pow(x.suc)) + sum(map(pow_list(p, x), von_mangoldt))
        sum(map(pow_list(p, x.suc), von_mangoldt)) =
            von_mangoldt(p.pow(x.suc)) + sum(map(pow_list(p, x), von_mangoldt))
        Nat.1 <= x.suc
        von_mangoldt_prime_power(p, x.suc)
        von_mangoldt(p.pow(x.suc)) = (from_nat[Real](p)).log.get_or_else(Real.0)
        sum(map(pow_list(p, x), von_mangoldt)) =
            from_nat[Real](x) * (from_nat[Real](p)).log.get_or_else(Real.0)
        sum(map(pow_list(p, x.suc), von_mangoldt)) =
            (from_nat[Real](p)).log.get_or_else(Real.0) +
                from_nat[Real](x) * (from_nat[Real](p)).log.get_or_else(Real.0)
        from_nat[Real](x.suc) = from_nat[Real](x) + Real.1
        (from_nat[Real](p)).log.get_or_else(Real.0) +
            from_nat[Real](x) * (from_nat[Real](p)).log.get_or_else(Real.0) =
            from_nat[Real](x.suc) * (from_nat[Real](p)).log.get_or_else(Real.0)
        sum(map(pow_list(p, x.suc), von_mangoldt)) =
            from_nat[Real](x.suc) * (from_nat[Real](p)).log.get_or_else(Real.0)
    }
}

/// The sum of `Lambda` over the descending powers of `p` is `k * log p`.
theorem pow_list_sum_von_mangoldt(p: Nat, k: Nat) {
    p.is_prime implies
        sum(map(pow_list(p, k), von_mangoldt)) =
            from_nat[Real](k) * (from_nat[Real](p)).log.get_or_else(Real.0)
} by {
    define pred(x: Nat) -> Bool {
        sum(map(pow_list(p, x), von_mangoldt)) =
            from_nat[Real](x) * (from_nat[Real](p)).log.get_or_else(Real.0)
    }
    if p.is_prime {
        pow_list_sum_von_mangoldt_zero(p)
        pred(Nat.0) = (sum(map(pow_list(p, Nat.0), von_mangoldt)) =
            from_nat[Real](Nat.0) * (from_nat[Real](p)).log.get_or_else(Real.0))
        pred(Nat.0)
        forall(x: Nat) {
            if pred(x) {
                pred(x) = (sum(map(pow_list(p, x), von_mangoldt)) =
                    from_nat[Real](x) * (from_nat[Real](p)).log.get_or_else(Real.0))
                pow_list_sum_von_mangoldt_suc(p, x)
                pred(x.suc) = (sum(map(pow_list(p, x.suc), von_mangoldt)) =
                    from_nat[Real](x.suc) * (from_nat[Real](p)).log.get_or_else(Real.0))
                pred(x.suc)
            }
        }
        pred(Nat.0) and forall(x: Nat) {
            pred(x) implies pred(x.suc)
        }
        Nat.induction(pred)
        forall(x: Nat) { pred(x) }
        pred(k)
    }
}

/// The logarithm of a natural power is the exponent times the logarithm.
theorem log_value_pow_nat(p: Nat, k: Nat) {
    p != Nat.0 implies
        (from_nat[Real](p.pow(k))).log.get_or_else(Real.0) =
            from_nat[Real](k) * (from_nat[Real](p)).log.get_or_else(Real.0)
} by {
    if p != Nat.0 {
        from_nat_real_pow(p, k)
        from_nat[Real](p.pow(k)) = from_nat[Real](p).pow(k)
        from_nat_real_pos_of_ne_zero(p)
        from_nat[Real](p) > Real.0
        log_some_of_pos_exists(from_nat[Real](p))
        let lp: Real satisfy {
            (from_nat[Real](p)).log = Option.some(lp)
        }
        option_get_or_else_some[Real](lp, Real.0)
        option_get_or_else(Option.some(lp), Real.0) = lp
        (from_nat[Real](p)).log.get_or_else(Real.0) = lp
        exp_log_or_zero(from_nat[Real](p), lp)
        lp.exp = from_nat[Real](p)
        ((from_nat[Real](p)).log.get_or_else(Real.0)).exp = from_nat[Real](p)
        exp_nat_mul((from_nat[Real](p)).log.get_or_else(Real.0), k)
        (from_nat[Real](k) * (from_nat[Real](p)).log.get_or_else(Real.0)).exp =
            ((from_nat[Real](p)).log.get_or_else(Real.0)).exp.pow(k)
        ((from_nat[Real](p)).log.get_or_else(Real.0)).exp.pow(k) = from_nat[Real](p).pow(k)
        (from_nat[Real](k) * (from_nat[Real](p)).log.get_or_else(Real.0)).exp =
            from_nat[Real](p).pow(k)
        exp_ne_zero(p, k)
        p.pow(k) != Nat.0
        from_nat_real_pos_of_ne_zero(p.pow(k))
        from_nat[Real](p.pow(k)) > Real.0
        from_nat[Real](p).pow(k) > Real.0
        log_some_of_pos_exists(from_nat[Real](p).pow(k))
        let lpk: Real satisfy {
            (from_nat[Real](p).pow(k)).log = Option.some(lpk)
        }
        option_get_or_else_some[Real](lpk, Real.0)
        option_get_or_else(Option.some(lpk), Real.0) = lpk
        (from_nat[Real](p).pow(k)).log.get_or_else(Real.0) = lpk
        exp_log_or_zero(from_nat[Real](p).pow(k), lpk)
        lpk.exp = from_nat[Real](p).pow(k)
        ((from_nat[Real](p).pow(k)).log.get_or_else(Real.0)).exp = from_nat[Real](p).pow(k)
        (from_nat[Real](k) * (from_nat[Real](p)).log.get_or_else(Real.0)).exp =
            ((from_nat[Real](p).pow(k)).log.get_or_else(Real.0)).exp
        exp_injective(from_nat[Real](k) * (from_nat[Real](p)).log.get_or_else(Real.0),
            (from_nat[Real](p).pow(k)).log.get_or_else(Real.0))
        from_nat[Real](k) * (from_nat[Real](p)).log.get_or_else(Real.0) =
            (from_nat[Real](p).pow(k)).log.get_or_else(Real.0)
        from_nat[Real](p.pow(k)) = from_nat[Real](p).pow(k)
        (from_nat[Real](p.pow(k))).log.get_or_else(Real.0) =
            from_nat[Real](k) * (from_nat[Real](p)).log.get_or_else(Real.0)
    }
}

/// The main identity: `sum_{d | p^k} Lambda(d) = (p^k).log` for a prime `p`.
theorem von_mangoldt_divisor_sum_prime_power(p: Nat, k: Nat) {
    p.is_prime implies
        sum(map(divisor_list(p.pow(k)), von_mangoldt)) =
            (from_nat[Real](p.pow(k))).log.get_or_else(Real.0)
} by {
    if p.is_prime {
        divisor_list_prime_power_sum_eq_pow_list_sum(p, k)
        sum(map(divisor_list(p.pow(k)), von_mangoldt)) =
            sum(map(pow_list(p, k), von_mangoldt))
        pow_list_sum_von_mangoldt(p, k)
        sum(map(pow_list(p, k), von_mangoldt)) =
            from_nat[Real](k) * (from_nat[Real](p)).log.get_or_else(Real.0)
        Nat.1 < p
        p != Nat.0
        log_value_pow_nat(p, k)
        (from_nat[Real](p.pow(k))).log.get_or_else(Real.0) =
            from_nat[Real](k) * (from_nat[Real](p)).log.get_or_else(Real.0)
        sum(map(divisor_list(p.pow(k)), von_mangoldt)) =
            (from_nat[Real](p.pow(k))).log.get_or_else(Real.0)
    }
}

/// The four positive divisors of a product of two distinct primes, in
/// descending order.
define prime_mul_divisors(p: Nat, q: Nat) -> List[Nat] {
    List.cons(p * q, List.cons(p, List.cons(q, List.cons(Nat.1, List.nil[Nat]))))
}

/// A product of two positive naturals is positive.
theorem nat_mul_pos_of_pos(a: Nat, b: Nat) {
    Nat.0 < a and Nat.0 < b implies Nat.0 < a * b
} by {
    if Nat.0 < a and Nat.0 < b {
        lt_imp_lte_suc(Nat.0, a)
        Nat.1 <= a
        lt_imp_lte_suc(Nat.0, b)
        Nat.1 <= b
        lte_mul_both(a, Nat.1, b)
        a * Nat.1 <= a * b
        mul_one_right(a)
        a * Nat.1 = a
        a <= a * b
        lte_trans(Nat.1, a, a * b)
        Nat.1 <= a * b
        lt_suc(Nat.0)
        Nat.0 < Nat.1
        lt_and_lte(Nat.0, Nat.1, a * b)
        Nat.0 < a * b
    }
}

/// Every positive divisor of a product of two distinct primes is one of
/// `1`, `p`, `q`, `p * q`.
theorem divisor_of_prime_mul(p: Nat, q: Nat, d: Nat) {
    p.is_prime and q.is_prime and p != q and Nat.0 < d and d.divides(p * q) implies
        prime_mul_divisors(p, q).contains(d)
} by {
    if p.is_prime and q.is_prime and p != q and Nat.0 < d and d.divides(p * q) {
        prime_mul_divisors(p, q) =
            List.cons(p * q, List.cons(p, List.cons(q, List.cons(Nat.1, List.nil[Nat]))))
        if d = Nat.1 {
            List.cons(q, List.cons(Nat.1, List.nil[Nat])).contains(Nat.1)
            List.cons(p, List.cons(q, List.cons(Nat.1, List.nil[Nat]))).contains(Nat.1)
            List.cons(p * q, List.cons(p, List.cons(q, List.cons(Nat.1, List.nil[Nat])))).contains(Nat.1)
            prime_mul_divisors(p, q).contains(d)
        } else {
            d != Nat.1
            Nat.1 < d
            has_prime_divisor(d)
            let r: Nat satisfy { r.is_prime and r.divides(d) }
            r.is_prime
            r.divides(d)
            divides_trans(r, d, p * q)
            r.divides(p * q)
            gcd_of_prime(r, p)
            r.gcd(p) = Nat.1 or r.divides(p)
            if r.divides(p) {
                prime_divisor_is_one_or_self(p, r)
                r = Nat.1 or r = p
                if r = Nat.1 {
                    Nat.1 < r
                    Nat.1 < Nat.1
                    lt_not_ref(Nat.1)
                    false
                }
                r = p
                p.divides(d)
                let c: Nat satisfy { p * c = d }
                p * c = d
                let b: Nat satisfy { d * b = p * q }
                d * b = p * q
                (p * c) * b = p * q
                p * (c * b) = p * q
                Nat.1 < p
                p != Nat.0
                mul_cancel_left(p, c * b, q)
                c * b = q
                prime_factor_dichotomy(q, c, b)
                (c = Nat.1 and b = q) or (c = q and b = Nat.1)
                if c = Nat.1 {
                    p * Nat.1 = d
                    p = d
                    d = p
                    List.cons(p, List.cons(q, List.cons(Nat.1, List.nil[Nat]))).contains(p)
                    List.cons(p * q, List.cons(p, List.cons(q, List.cons(Nat.1, List.nil[Nat])))).contains(p)
                    prime_mul_divisors(p, q).contains(d)
                } else {
                    c = q
                    p * q = d
                    d = p * q
                    List.cons(p * q, List.cons(p, List.cons(q, List.cons(Nat.1, List.nil[Nat])))).contains(p * q)
                    prime_mul_divisors(p, q).contains(d)
                }
            } else {
                r.gcd(p) = Nat.1
                r.coprime(p)
                coprime_divides_of_divides_mul(r, p, q)
                r.divides(q)
                prime_divisor_is_one_or_self(q, r)
                r = Nat.1 or r = q
                if r = Nat.1 {
                    Nat.1 < r
                    Nat.1 < Nat.1
                    lt_not_ref(Nat.1)
                    false
                }
                r = q
                q.divides(d)
                let c: Nat satisfy { q * c = d }
                q * c = d
                let b: Nat satisfy { d * b = p * q }
                d * b = p * q
                (q * c) * b = p * q
                q * (c * b) = p * q
                Nat.1 < q
                q != Nat.0
                mul_cancel_left(q, c * b, p)
                c * b = p
                prime_factor_dichotomy(p, c, b)
                (c = Nat.1 and b = p) or (c = p and b = Nat.1)
                if c = Nat.1 {
                    q * Nat.1 = d
                    q = d
                    d = q
                    List.cons(q, List.cons(Nat.1, List.nil[Nat])).contains(q)
                    List.cons(p, List.cons(q, List.cons(Nat.1, List.nil[Nat]))).contains(q)
                    List.cons(p * q, List.cons(p, List.cons(q, List.cons(Nat.1, List.nil[Nat])))).contains(q)
                    prime_mul_divisors(p, q).contains(d)
                } else {
                    c = p
                    q * p = d
                    p * q = d
                    d = p * q
                    List.cons(p * q, List.cons(p, List.cons(q, List.cons(Nat.1, List.nil[Nat])))).contains(p * q)
                    prime_mul_divisors(p, q).contains(d)
                }
            }
            prime_mul_divisors(p, q).contains(d)
        }
        prime_mul_divisors(p, q).contains(d)
    }
}
/// A divisor of `p * q` is one of `1`, `p`, `q`, `p * q`.
theorem divisor_list_prime_mul_contains_imp(p: Nat, q: Nat, d: Nat) {
    p.is_prime and q.is_prime and p != q and divisor_list(p * q).contains(d)
        implies prime_mul_divisors(p, q).contains(d)
} by {
    if p.is_prime and q.is_prime and p != q and divisor_list(p * q).contains(d) {
        divisor_list_contains_implies(p * q, d)
        Nat.0 < d and d.divides(p * q)
        divisor_of_prime_mul(p, q, d)
        prime_mul_divisors(p, q).contains(d)
    }
}
/// One of `1`, `p`, `q`, `p * q` is a divisor of `p * q`.
theorem prime_mul_contains_imp_divisor_list(p: Nat, q: Nat, d: Nat) {
    p.is_prime and q.is_prime and p != q and prime_mul_divisors(p, q).contains(d)
        implies divisor_list(p * q).contains(d)
} by {
    if p.is_prime and q.is_prime and p != q and prime_mul_divisors(p, q).contains(d) {
        prime_mul_divisors(p, q) =
            List.cons(p * q, List.cons(p, List.cons(q, List.cons(Nat.1, List.nil[Nat]))))
        List.cons(p * q, List.cons(p, List.cons(q, List.cons(Nat.1, List.nil[Nat])))).contains(d)
        Nat.1 < p
        p != Nat.0
        Nat.1 < q
        q != Nat.0
        pos_of_ne_zero(p)
        Nat.0 < p
        pos_of_ne_zero(q)
        Nat.0 < q
        nat_mul_pos_of_pos(p, q)
        Nat.0 < p * q
        if d = p * q {
            divides_self(p * q)
            d.divides(p * q)
            Nat.0 < d
            divisor_list_contains_of(p * q, d)
            divisor_list(p * q).contains(d)
        } else {
            d != p * q
            List.cons(p, List.cons(q, List.cons(Nat.1, List.nil[Nat]))).contains(d)
            if d = p {
                p.divides(p * q)
                d.divides(p * q)
                Nat.0 < d
                divisor_list_contains_of(p * q, d)
                divisor_list(p * q).contains(d)
            } else {
                d != p
                List.cons(q, List.cons(Nat.1, List.nil[Nat])).contains(d)
                if d = q {
                    q.divides(p * q)
                    d.divides(p * q)
                    Nat.0 < d
                    divisor_list_contains_of(p * q, d)
                    divisor_list(p * q).contains(d)
                } else {
                    d != q
                    List.cons(Nat.1, List.nil[Nat]).contains(d)
                    d = Nat.1 or List.nil[Nat].contains(d)
                    not List.nil[Nat].contains(d)
                    d = Nat.1
                    d.divides(p * q)
                    Nat.0 < d
                    divisor_list_contains_of(p * q, d)
                    divisor_list(p * q).contains(d)
                }
            }
        }
        divisor_list(p * q).contains(d)
    }
}

/// The explicit four-divisor list of a product of two distinct primes is unique.
theorem prime_mul_divisors_unique(p: Nat, q: Nat) {
    p.is_prime and q.is_prime and p != q implies prime_mul_divisors(p, q).is_unique
} by {
    if p.is_prime and q.is_prime and p != q {
        prime_mul_divisors(p, q) =
            List.cons(p * q, List.cons(p, List.cons(q, List.cons(Nat.1, List.nil[Nat]))))
        List.cons(Nat.1, List.nil[Nat]).is_unique
        if List.cons(Nat.1, List.nil[Nat]).contains(q) {
            q = Nat.1 or List.nil[Nat].contains(q)
            not List.nil[Nat].contains(q)
            q = Nat.1
            Nat.1 < q
            Nat.1 < Nat.1
            lt_not_ref(Nat.1)
            false
        }
        not List.cons(Nat.1, List.nil[Nat]).contains(q)
        cons_unique_of_tail_unique_not_contains(q, List.cons(Nat.1, List.nil[Nat]))
        List.cons(q, List.cons(Nat.1, List.nil[Nat])).is_unique
        if List.cons(q, List.cons(Nat.1, List.nil[Nat])).contains(p) {
            p = q or List.cons(Nat.1, List.nil[Nat]).contains(p)
            if p = q {
                p != q
                false
            }
            List.cons(Nat.1, List.nil[Nat]).contains(p)
            p = Nat.1 or List.nil[Nat].contains(p)
            not List.nil[Nat].contains(p)
            p = Nat.1
            Nat.1 < p
            Nat.1 < Nat.1
            lt_not_ref(Nat.1)
            false
        }
        not List.cons(q, List.cons(Nat.1, List.nil[Nat])).contains(p)
        cons_unique_of_tail_unique_not_contains(p, List.cons(q, List.cons(Nat.1, List.nil[Nat])))
        List.cons(p, List.cons(q, List.cons(Nat.1, List.nil[Nat]))).is_unique
        if List.cons(p, List.cons(q, List.cons(Nat.1, List.nil[Nat]))).contains(p * q) {
            p * q = p or List.cons(q, List.cons(Nat.1, List.nil[Nat])).contains(p * q)
            if p * q = p {
                p * q = p * Nat.1
                Nat.1 < p
                p != Nat.0
                mul_cancel_left(p, q, Nat.1)
                q = Nat.1
                Nat.1 < q
                Nat.1 < Nat.1
                lt_not_ref(Nat.1)
                false
            }
            List.cons(q, List.cons(Nat.1, List.nil[Nat])).contains(p * q)
            p * q = q or List.cons(Nat.1, List.nil[Nat]).contains(p * q)
            if p * q = q {
                q * p = q * Nat.1
                Nat.1 < q
                q != Nat.0
                mul_cancel_left(q, p, Nat.1)
                p = Nat.1
                Nat.1 < p
                Nat.1 < Nat.1
                lt_not_ref(Nat.1)
                false
            }
            List.cons(Nat.1, List.nil[Nat]).contains(p * q)
            p * q = Nat.1 or List.nil[Nat].contains(p * q)
            not List.nil[Nat].contains(p * q)
            p * q = Nat.1
            p.divides(Nat.1)
            prime_does_not_divide_one(p)
            not p.divides(Nat.1)
            false
        }
        not List.cons(p, List.cons(q, List.cons(Nat.1, List.nil[Nat]))).contains(p * q)
        cons_unique_of_tail_unique_not_contains(p * q, List.cons(p, List.cons(q, List.cons(Nat.1, List.nil[Nat]))))
        List.cons(p * q, List.cons(p, List.cons(q, List.cons(Nat.1, List.nil[Nat])))).is_unique
        prime_mul_divisors(p, q).is_unique
    }
}

/// The divisor list of `p * q` and the explicit four-divisor list carry the
/// same members, so the divisor sums of `Lambda` agree.
theorem divisor_list_prime_mul_sum_eq_list_sum(p: Nat, q: Nat) {
    p.is_prime and q.is_prime and p != q implies
        sum(map(divisor_list(p * q), von_mangoldt)) =
            sum(map(prime_mul_divisors(p, q), von_mangoldt))
} by {
    if p.is_prime and q.is_prime and p != q {
        divisor_list_is_unique(p * q)
        divisor_list(p * q).is_unique
        prime_mul_divisors(p, q) =
            List.cons(p * q, List.cons(p, List.cons(q, List.cons(Nat.1, List.nil[Nat]))))
        List.cons(p * q, List.cons(p, List.cons(q, List.cons(Nat.1, List.nil[Nat])))).is_unique
        prime_mul_divisors(p, q).is_unique
        forall(d: Nat) {
            if divisor_list(p * q).contains(d) {
                divisor_list_prime_mul_contains_imp(p, q, d)
                prime_mul_divisors(p, q).contains(d)
                divisor_list(p * q).contains(d) = prime_mul_divisors(p, q).contains(d)
            }
            if not divisor_list(p * q).contains(d) {
                if prime_mul_divisors(p, q).contains(d) {
                    prime_mul_contains_imp_divisor_list(p, q, d)
                    divisor_list(p * q).contains(d)
                    not divisor_list(p * q).contains(d)
                    false
                }
                not prime_mul_divisors(p, q).contains(d)
                divisor_list(p * q).contains(d) = prime_mul_divisors(p, q).contains(d)
            }
            divisor_list(p * q).contains(d) = prime_mul_divisors(p, q).contains(d)
        }
        unique_same_contains_map_sum_eq(
            divisor_list(p * q), prime_mul_divisors(p, q), von_mangoldt)
        sum(map(divisor_list(p * q), von_mangoldt)) =
            sum(map(prime_mul_divisors(p, q), von_mangoldt))
    }
}

/// The sum of `Lambda` over `[p * q, p, q, 1]` is `log p + log q`.
theorem prime_mul_list_sum_von_mangoldt(p: Nat, q: Nat) {
    p.is_prime and q.is_prime and p != q implies
        sum(map(prime_mul_divisors(p, q), von_mangoldt)) =
            (from_nat[Real](p)).log.get_or_else(Real.0) + (from_nat[Real](q)).log.get_or_else(Real.0)
} by {
    if p.is_prime and q.is_prime and p != q {
        prime_mul_divisors(p, q) =
            List.cons(p * q, List.cons(p, List.cons(q, List.cons(Nat.1, List.nil[Nat]))))
        sum(map(prime_mul_divisors(p, q), von_mangoldt)) =
            sum(map(List.cons(p * q, List.cons(p, List.cons(q, List.cons(Nat.1, List.nil[Nat])))), von_mangoldt))
        map(List.cons(p * q, List.cons(p, List.cons(q, List.cons(Nat.1, List.nil[Nat])))), von_mangoldt) =
            List.cons(von_mangoldt(p * q), map(List.cons(p, List.cons(q, List.cons(Nat.1, List.nil[Nat]))), von_mangoldt))
        sum(List.cons(von_mangoldt(p * q), map(List.cons(p, List.cons(q, List.cons(Nat.1, List.nil[Nat]))), von_mangoldt))) =
            von_mangoldt(p * q) + sum(map(List.cons(p, List.cons(q, List.cons(Nat.1, List.nil[Nat]))), von_mangoldt))
        sum(map(List.cons(p * q, List.cons(p, List.cons(q, List.cons(Nat.1, List.nil[Nat])))), von_mangoldt)) =
            von_mangoldt(p * q) + sum(map(List.cons(p, List.cons(q, List.cons(Nat.1, List.nil[Nat]))), von_mangoldt))
        map(List.cons(p, List.cons(q, List.cons(Nat.1, List.nil[Nat]))), von_mangoldt) =
            List.cons(von_mangoldt(p), map(List.cons(q, List.cons(Nat.1, List.nil[Nat])), von_mangoldt))
        sum(List.cons(von_mangoldt(p), map(List.cons(q, List.cons(Nat.1, List.nil[Nat])), von_mangoldt))) =
            von_mangoldt(p) + sum(map(List.cons(q, List.cons(Nat.1, List.nil[Nat])), von_mangoldt))
        sum(map(List.cons(p, List.cons(q, List.cons(Nat.1, List.nil[Nat]))), von_mangoldt)) =
            von_mangoldt(p) + sum(map(List.cons(q, List.cons(Nat.1, List.nil[Nat])), von_mangoldt))
        map(List.cons(q, List.cons(Nat.1, List.nil[Nat])), von_mangoldt) =
            List.cons(von_mangoldt(q), map(List.cons(Nat.1, List.nil[Nat]), von_mangoldt))
        sum(List.cons(von_mangoldt(q), map(List.cons(Nat.1, List.nil[Nat]), von_mangoldt))) =
            von_mangoldt(q) + sum(map(List.cons(Nat.1, List.nil[Nat]), von_mangoldt))
        sum(map(List.cons(q, List.cons(Nat.1, List.nil[Nat])), von_mangoldt)) =
            von_mangoldt(q) + sum(map(List.cons(Nat.1, List.nil[Nat]), von_mangoldt))
        map(List.cons(Nat.1, List.nil[Nat]), von_mangoldt) =
            List.cons(von_mangoldt(Nat.1), map(List.nil[Nat], von_mangoldt))
        map(List.nil[Nat], von_mangoldt) = List.nil[Real]
        map(List.cons(Nat.1, List.nil[Nat]), von_mangoldt) =
            List.cons(von_mangoldt(Nat.1), List.nil[Real])
        sum(List.cons(von_mangoldt(Nat.1), List.nil[Real])) =
            von_mangoldt(Nat.1) + sum(List.nil[Real])
        sum(map(List.cons(Nat.1, List.nil[Nat]), von_mangoldt)) =
            von_mangoldt(Nat.1) + sum(map(List.nil[Nat], von_mangoldt))
        sum(map(List.nil[Nat], von_mangoldt)) = Real.0
        sum(map(prime_mul_divisors(p, q), von_mangoldt)) =
            von_mangoldt(p * q) + (von_mangoldt(p) + (von_mangoldt(q) + (von_mangoldt(Nat.1) + Real.0)))
        von_mangoldt_prime_mul(p, q)
        von_mangoldt(p * q) = Real.0
        von_mangoldt_prime_power(p, Nat.1)
        von_mangoldt(p.pow(Nat.1)) = (from_nat[Real](p)).log.get_or_else(Real.0)
        p.pow(Nat.1) = p
        von_mangoldt(p) = (from_nat[Real](p)).log.get_or_else(Real.0)
        von_mangoldt_prime_power(q, Nat.1)
        von_mangoldt(q.pow(Nat.1)) = (from_nat[Real](q)).log.get_or_else(Real.0)
        q.pow(Nat.1) = q
        von_mangoldt(q) = (from_nat[Real](q)).log.get_or_else(Real.0)
        von_mangoldt_one
        von_mangoldt(Nat.1) = Real.0
        sum(map(prime_mul_divisors(p, q), von_mangoldt)) =
            (from_nat[Real](p)).log.get_or_else(Real.0) + (from_nat[Real](q)).log.get_or_else(Real.0)
    }
}

/// The logarithm of a product of two naturals is the sum of the logarithms.
theorem log_value_mul_nat(p: Nat, q: Nat) {
    p != Nat.0 and q != Nat.0 implies
        (from_nat[Real](p * q)).log.get_or_else(Real.0) =
            (from_nat[Real](p)).log.get_or_else(Real.0) + (from_nat[Real](q)).log.get_or_else(Real.0)
} by {
    if p != Nat.0 and q != Nat.0 {
        from_nat_mul[Real](p, q)
        from_nat[Real](p * q) = from_nat[Real](p) * from_nat[Real](q)
        from_nat_real_pos_of_ne_zero(p)
        from_nat[Real](p) > Real.0
        from_nat_real_pos_of_ne_zero(q)
        from_nat[Real](q) > Real.0
        log_some_of_pos_exists(from_nat[Real](p))
        let lp: Real satisfy {
            (from_nat[Real](p)).log = Option.some(lp)
        }
        option_get_or_else_some[Real](lp, Real.0)
        option_get_or_else(Option.some(lp), Real.0) = lp
        (from_nat[Real](p)).log.get_or_else(Real.0) = lp
        log_some_of_pos_exists(from_nat[Real](q))
        let lq: Real satisfy {
            (from_nat[Real](q)).log = Option.some(lq)
        }
        option_get_or_else_some[Real](lq, Real.0)
        option_get_or_else(Option.some(lq), Real.0) = lq
        (from_nat[Real](q)).log.get_or_else(Real.0) = lq
        log_mul(from_nat[Real](p), from_nat[Real](q), lp, lq)
        (from_nat[Real](p) * from_nat[Real](q)).log = Option.some(lp + lq)
        lp + lq = (from_nat[Real](p)).log.get_or_else(Real.0) + (from_nat[Real](q)).log.get_or_else(Real.0)
        (from_nat[Real](p) * from_nat[Real](q)).log =
            Option.some((from_nat[Real](p)).log.get_or_else(Real.0) + (from_nat[Real](q)).log.get_or_else(Real.0))
        if p * q = Nat.0 {
            mul_to_zero(p, q)
            p = Nat.0 or q = Nat.0
            if p = Nat.0 {
                p != Nat.0
                false
            }
            q = Nat.0
            q != Nat.0
            false
        }
        p * q != Nat.0
        from_nat_real_pos_of_ne_zero(p * q)
        from_nat[Real](p * q) > Real.0
        from_nat[Real](p) * from_nat[Real](q) > Real.0
        log_some_of_pos_exists(from_nat[Real](p) * from_nat[Real](q))
        let lpq: Real satisfy {
            (from_nat[Real](p) * from_nat[Real](q)).log = Option.some(lpq)
        }
        option_get_or_else_some[Real](lpq, Real.0)
        option_get_or_else(Option.some(lpq), Real.0) = lpq
        (from_nat[Real](p) * from_nat[Real](q)).log.get_or_else(Real.0) = lpq
        (from_nat[Real](p) * from_nat[Real](q)).log =
            Option.some((from_nat[Real](p) * from_nat[Real](q)).log.get_or_else(Real.0))
        Option.some((from_nat[Real](p) * from_nat[Real](q)).log.get_or_else(Real.0)) =
            Option.some((from_nat[Real](p)).log.get_or_else(Real.0) + (from_nat[Real](q)).log.get_or_else(Real.0))
        some_injective[Real]((from_nat[Real](p) * from_nat[Real](q)).log.get_or_else(Real.0),
            (from_nat[Real](p)).log.get_or_else(Real.0) + (from_nat[Real](q)).log.get_or_else(Real.0))
        (from_nat[Real](p) * from_nat[Real](q)).log.get_or_else(Real.0) =
            (from_nat[Real](p)).log.get_or_else(Real.0) + (from_nat[Real](q)).log.get_or_else(Real.0)
        (from_nat[Real](p * q)).log.get_or_else(Real.0) =
            (from_nat[Real](p)).log.get_or_else(Real.0) + (from_nat[Real](q)).log.get_or_else(Real.0)
    }
}

/// The main identity for a product of two distinct primes:
/// `sum_{d | p * q} Lambda(d) = (p * q).log`.
theorem von_mangoldt_divisor_sum_prime_mul(p: Nat, q: Nat) {
    p.is_prime and q.is_prime and p != q implies
        sum(map(divisor_list(p * q), von_mangoldt)) =
            (from_nat[Real](p * q)).log.get_or_else(Real.0)
} by {
    if p.is_prime and q.is_prime and p != q {
        divisor_list_prime_mul_sum_eq_list_sum(p, q)
        sum(map(divisor_list(p * q), von_mangoldt)) =
            sum(map(prime_mul_divisors(p, q), von_mangoldt))
        prime_mul_list_sum_von_mangoldt(p, q)
        sum(map(prime_mul_divisors(p, q), von_mangoldt)) =
            (from_nat[Real](p)).log.get_or_else(Real.0) + (from_nat[Real](q)).log.get_or_else(Real.0)
        Nat.1 < p
        p != Nat.0
        Nat.1 < q
        q != Nat.0
        log_value_mul_nat(p, q)
        (from_nat[Real](p * q)).log.get_or_else(Real.0) =
            (from_nat[Real](p)).log.get_or_else(Real.0) + (from_nat[Real](q)).log.get_or_else(Real.0)
        sum(map(divisor_list(p * q), von_mangoldt)) =
            (from_nat[Real](p * q)).log.get_or_else(Real.0)
    }
}

// The general identity: for every positive integer `n`,
//     sum_{d | n} Lambda(d) = log n.
// The library's divisor list of a general `n` is not a prime-power list, so
// the proof needs the prime factorisation `n = p_1^{k_1} ... p_r^{k_r}`:
// every divisor of `n` is a product of divisors of the prime powers, the
// von Mangoldt function is supported on prime powers only, and the divisor
// sum factors as the sum of the prime-power sums proved above
// (`von_mangoldt_divisor_sum_prime_power` and
// `von_mangoldt_divisor_sum_prime_mul`), giving `sum_i k_i log p_i = log n`
// by the logarithm product law.  The statement is recorded here for future
// work on the prime number theorem.
// theorem von_mangoldt_divisor_sum(n: Nat) {
//     Nat.0 < n implies
//         sum(map(divisor_list(n), von_mangoldt)) =
//             (from_nat[Real](n)).log.get_or_else(Real.0)
// }
