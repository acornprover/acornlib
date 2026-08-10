/// Fermat's little theorem and its classical consequences.
///
/// This file gathers the standard consequences of Fermat's little theorem in
/// one place: the unconditional form `a^p ≡ a (mod p)` (with a case-split
/// proof), the base-two form `2^(p - 1) ≡ 1 (mod p)` for odd primes, the
/// converse direction of the Fermat test (a classical primality criterion),
/// the divisibility of the multiplicative order modulo a prime by `p - 1`,
/// and Wilson's theorem.
///
/// Each theorem below is proved in its home file — `fermat.ac` (Fermat),
/// `totient.ac` (Euler), `multiplicative_order.ac` (order machinery),
/// `wilson.ac` (Wilson) — and is either restated here with a citation or
/// proved from those building blocks.

from nat import Nat
from nat import divides_self, divides_mul, divides_mod, small_mod, divisor_lt,
    lt_trans, zero_exp, exp_add, exp_one
from number_theory.congruence import congr_mod_refl, congr_mod_symm,
    congr_mod_trans, congr_mod_mul, congr_mod_pow, congr_mod_zero_of_divides
from number_theory.coprime import nat_divides_one_imp_one
from number_theory.fermat import fermats_little_congr, fermat_euler
from number_theory.totient import not_coprime_imp_divides_prime,
    coprime_below_prime, totient_prime
from number_theory.wilson import one_le_sub_one_of_one_lt,
    sub_one_add_one_eq_self, prime_imp_wilson_factorial_congr,
    wilson_factorial_congr_imp_prime
from number_theory.multiplicative_order import multiplicative_order_mod,
    multiplicative_order_mod_divides_totient, multiplicative_order_mod_le_totient
numerals Nat

// ---------------------------------------------------------------------------
// Fermat's little theorem, unconditional form: a^p ≡ a (mod p).
// ---------------------------------------------------------------------------

/// Fermat's little theorem for an arbitrary base: for a prime `p` and any
/// natural `a`, the power `a^p` is congruent to `a` modulo `p`. This holds
/// for every `a` without any coprimality hypothesis. This is
/// `fermats_little_congr` from fermat.ac, restated.
theorem fermat_unconditional(p: Nat, a: Nat) {
    p.is_prime implies a.pow(p).congr_mod(a, p)
} by {
    if p.is_prime {
        fermats_little_congr(p, a)
        a.pow(p).congr_mod(a, p)
    }
}

/// The same statement, proved by splitting on whether `p` divides `a`: when
/// `p | a`, both sides vanish modulo `p`; otherwise `a` is coprime to `p`
/// and Fermat-Euler gives `a^(p - 1) ≡ 1 (mod p)`, which multiplies by `a`
/// to the claim.
theorem fermat_unconditional_by_cases(p: Nat, a: Nat) {
    p.is_prime implies a.pow(p).congr_mod(a, p)
} by {
    if p.is_prime {
        if p.divides(a) {
            congr_mod_zero_of_divides(p, a)
            a.congr_mod(Nat.0, p)
            congr_mod_pow(a, Nat.0, p, p)
            a.pow(p).congr_mod(Nat.0.pow(p), p)
            Nat.1 < p
            p != Nat.0
            zero_exp(p)
            Nat.0.pow(p) = Nat.0
            a.pow(p).congr_mod(Nat.0, p)
            congr_mod_symm(a, Nat.0, p)
            Nat.0.congr_mod(a, p)
            congr_mod_trans(a.pow(p), Nat.0, a, p)
            a.pow(p).congr_mod(a, p)
        } else {
            if not a.coprime(p) {
                not_coprime_imp_divides_prime(p, a)
                p.divides(a)
                false
            }
            a.coprime(p)
            fermat_euler(p, a)
            a.pow(p - Nat.1).congr_mod(Nat.1, p)
            congr_mod_refl(a, p)
            a.congr_mod(a, p)
            congr_mod_mul(a.pow(p - Nat.1), a, Nat.1, a, p)
            (a.pow(p - Nat.1) * a).congr_mod(Nat.1 * a, p)
            Nat.1 * a = a
            (a.pow(p - Nat.1) * a).congr_mod(a, p)
            Nat.1 < p
            p != Nat.0
            sub_one_add_one_eq_self(p)
            (p - Nat.1) + Nat.1 = p
            exp_add(a, p - Nat.1, Nat.1)
            a.pow((p - Nat.1) + Nat.1) = a.pow(p - Nat.1) * a.pow(Nat.1)
            a.pow(p) = a.pow(p - Nat.1) * a.pow(Nat.1)
            exp_one(a)
            a.pow(Nat.1) = a
            a.pow(p) = a.pow(p - Nat.1) * a
            a.pow(p).congr_mod(a, p)
        }
    }
}

// ---------------------------------------------------------------------------
// Base two.
// ---------------------------------------------------------------------------

/// Fermat's little theorem for base two: for an odd prime `p` (that is,
/// `p` prime with `2 < p`), the power `2^(p - 1)` is congruent to `1`
/// modulo `p`. An odd prime does not divide `2`, so this is the unit case
/// `fermat_euler` from fermat.ac applied at `a = 2`.
theorem fermat_base_two_odd_prime(p: Nat) {
    p.is_prime and Nat.2 < p implies Nat.2.pow(p - Nat.1).congr_mod(Nat.1, p)
} by {
    if p.is_prime and Nat.2 < p {
        Nat.1 <= Nat.2
        coprime_below_prime(p, Nat.2)
        Nat.2.coprime(p)
        fermat_euler(p, Nat.2)
        Nat.2.pow(p - Nat.1).congr_mod(Nat.1, p)
    }
}

// ---------------------------------------------------------------------------
// The converse direction of the Fermat test.
// ---------------------------------------------------------------------------

/// True if every base `a` in `[1, p)` passes the Fermat test for `p`: the
/// congruence `a^(p - 1) ≡ 1 (mod p)`.
define fermat_test_passes(p: Nat) -> Bool {
    forall(a: Nat) { Nat.1 <= a and a < p implies a.pow(p - Nat.1).congr_mod(Nat.1, p) }
}

/// A modulus passing the Fermat test applies to any base in `[1, p)`.
theorem fermat_test_passes_apply(p: Nat, a: Nat) {
    fermat_test_passes(p) and Nat.1 <= a and a < p
        implies a.pow(p - Nat.1).congr_mod(Nat.1, p)
} by {
    if fermat_test_passes(p) and Nat.1 <= a and a < p {
        fermat_test_passes(p) = forall(x: Nat) {
            Nat.1 <= x and x < p implies x.pow(p - Nat.1).congr_mod(Nat.1, p)
        }
        forall(x: Nat) {
            Nat.1 <= x and x < p implies x.pow(p - Nat.1).congr_mod(Nat.1, p)
        }
        a.pow(p - Nat.1).congr_mod(Nat.1, p)
    }
}

/// A pointwise Fermat test over every base in `[1, p)` is the passing
/// predicate.
theorem fermat_test_passes_intro(p: Nat) {
    (forall(a: Nat) { Nat.1 <= a and a < p implies a.pow(p - Nat.1).congr_mod(Nat.1, p) })
        implies fermat_test_passes(p)
} by {
    if forall(a: Nat) { Nat.1 <= a and a < p implies a.pow(p - Nat.1).congr_mod(Nat.1, p) } {
        fermat_test_passes(p) = forall(a: Nat) {
            Nat.1 <= a and a < p implies a.pow(p - Nat.1).congr_mod(Nat.1, p)
        }
        fermat_test_passes(p)
    }
}

/// A positive base divides every power of itself with positive exponent.
theorem divides_pow_self(b: Nat, k: Nat) {
    Nat.1 <= k implies b.divides(b.pow(k))
} by {
    if Nat.1 <= k {
        Nat.0 < k
        k != Nat.0
        let kp: Nat satisfy { kp.suc = k }
        kp.suc = kp + Nat.1
        kp + Nat.1 = k
        exp_add(b, kp, Nat.1)
        b.pow(kp + Nat.1) = b.pow(kp) * b.pow(Nat.1)
        b.pow(k) = b.pow(kp) * b.pow(Nat.1)
        exp_one(b)
        b.pow(Nat.1) = b
        b.pow(k) = b.pow(kp) * b
        divides_self(b)
        b.divides(b)
        divides_mul(b, b.pow(kp), b)
        b.divides(b * b.pow(kp))
        b * b.pow(kp) = b.pow(kp) * b
        b.divides(b.pow(k))
    }
}

/// A composite modulus cannot pass the Fermat test for every base in
/// `[1, p)`. If `p = b * c` with `1 < b`, then the base `b` satisfies
/// `b | b^(p - 1)` and `b | p`, while the test congruence forces `b | 1`, a
/// contradiction.
theorem composite_not_fermat_test(p: Nat) {
    p.is_composite implies not fermat_test_passes(p)
} by {
    if p.is_composite {
        let (b: Nat, c: Nat) satisfy {
            Nat.1 < b and Nat.1 < c and p = b * c
        }
        b != Nat.0
        b.divides(p)
        divisor_lt(b, c, p)
        b < p
        lt_trans(Nat.1, b, p)
        Nat.1 < p
        p != Nat.0
        one_le_sub_one_of_one_lt(p)
        Nat.1 <= p - Nat.1
        divides_pow_self(b, p - Nat.1)
        b.divides(b.pow(p - Nat.1))
        if fermat_test_passes(p) {
            fermat_test_passes_apply(p, b)
            Nat.1 <= b
            b < p
            b.pow(p - Nat.1).congr_mod(Nat.1, p)
            b.pow(p - Nat.1).mod(p) = Nat.1.mod(p)
            divides_mod(b.pow(p - Nat.1), p, b)
            b.divides(b.pow(p - Nat.1).mod(p))
            small_mod(Nat.1, p)
            Nat.1.mod(p) = Nat.1
            b.divides(Nat.1)
            nat_divides_one_imp_one(b)
            b = Nat.1
            false
        }
    }
}

/// The converse direction of the Fermat test: if every base `a` with
/// `1 <= a < p` satisfies `a^(p - 1) ≡ 1 (mod p)`, then `p` is prime.
///
/// Note: the version that only assumes the congruence for bases coprime to
/// `p` is false in general — composite `p` satisfying it for all coprime
/// bases are the Carmichael numbers (e.g. 561) — so the hypothesis ranges
/// over every base below `p`.
theorem fermat_test_converse(p: Nat) {
    Nat.1 < p and fermat_test_passes(p) implies p.is_prime
} by {
    if Nat.1 < p and fermat_test_passes(p) {
        if not p.is_prime {
            p.is_composite
            composite_not_fermat_test(p)
            not fermat_test_passes(p)
            false
        }
    }
}

// ---------------------------------------------------------------------------
// The multiplicative order modulo a prime.
// ---------------------------------------------------------------------------

/// The multiplicative order of `a` modulo a prime `p` divides `p - 1`, for
/// `a` coprime to `p`. Euler's theorem gives `a^(p - 1) ≡ 1 (mod p)`, so the
/// order — the least positive exponent reaching `1` — divides `p - 1`.
theorem order_mod_prime_divides_pred(p: Nat, a: Nat) {
    p.is_prime and a.coprime(p) implies multiplicative_order_mod(a, p).divides(p - Nat.1)
} by {
    if p.is_prime and a.coprime(p) {
        Nat.1 < p
        p != Nat.0
        multiplicative_order_mod_divides_totient(a, p)
        multiplicative_order_mod(a, p).divides(p.totient)
        totient_prime(p)
        p.totient = p - Nat.1
        multiplicative_order_mod(a, p).divides(p - Nat.1)
    }
}

/// The multiplicative order of `a` modulo a prime `p` is at most `p - 1`.
theorem order_mod_prime_le_pred(p: Nat, a: Nat) {
    p.is_prime and a.coprime(p) implies multiplicative_order_mod(a, p) <= p - Nat.1
} by {
    if p.is_prime and a.coprime(p) {
        Nat.1 < p
        p != Nat.0
        multiplicative_order_mod_le_totient(a, p)
        multiplicative_order_mod(a, p) <= p.totient
        totient_prime(p)
        p.totient = p - Nat.1
        multiplicative_order_mod(a, p) <= p - Nat.1
    }
}

// ---------------------------------------------------------------------------
// Wilson's theorem.
// ---------------------------------------------------------------------------

/// Wilson's theorem: for a prime `p`, the factorial `(p - 1)!` is congruent
/// to `p - 1` modulo `p`. Since `p - 1` is the canonical representative of
/// `-1` modulo `p`, this is the classical statement `(p - 1)! ≡ -1 (mod p)`.
/// This is `prime_imp_wilson_factorial_congr` from wilson.ac, restated.
theorem wilsons_factorial_congr_prime(p: Nat) {
    p.is_prime implies (p - Nat.1).factorial.congr_mod(p - Nat.1, p)
} by {
    if p.is_prime {
        prime_imp_wilson_factorial_congr(p)
        (p - Nat.1).factorial.congr_mod(p - Nat.1, p)
    }
}

/// Wilson's theorem in both directions: `p` is prime exactly when `p != 1`
/// and `(p - 1)! ≡ p - 1 (mod p)`. Combines
/// `prime_imp_wilson_factorial_congr` and `wilson_factorial_congr_imp_prime`
/// from wilson.ac.
theorem wilson_iff_prime(p: Nat) {
    p.is_prime = (p != Nat.1 and (p - Nat.1).factorial.congr_mod(p - Nat.1, p))
} by {
    if p.is_prime {
        p != Nat.1
        wilsons_factorial_congr_prime(p)
        (p - Nat.1).factorial.congr_mod(p - Nat.1, p)
        p != Nat.1 and (p - Nat.1).factorial.congr_mod(p - Nat.1, p)
    }
    if p != Nat.1 and (p - Nat.1).factorial.congr_mod(p - Nat.1, p) {
        wilson_factorial_congr_imp_prime(p)
        p.is_prime
    }
}
