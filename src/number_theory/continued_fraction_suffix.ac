from nat import Nat
from rat import Rat
from list import List, drop_zero, drop_one, drop_twice, tail_cancels_cons
from data.list.list_prefix_suffix import list_drop_nil, list_drop_cons_suc,
    list_drop_length_nil
from number_theory.continued_fraction import finite_continued_fraction_coefficients,
    ContinuedFraction, continued_fraction_coefficients_valid,
    positive_continued_fraction_tail, positive_continued_fraction_tail_nil,
    continued_fraction_value, continued_fraction_value_nil, continued_fraction_value_cons,
    continuant, continuant_nil, continued_fraction_numerator,
    continued_fraction_numerator_nil, continued_fraction_denominator,
    continued_fraction_denominator_nil, continued_fraction_convergent,
    continued_fraction_convergent_nil,
    continued_fraction_denominator_positive_of_valid_coefficients,
    continued_fraction_denominator_ne_zero_of_valid_coefficients,
    continued_fraction_convergent_first_eq_numerator,
    continued_fraction_convergent_second_eq_denominator,
    continued_fraction_convergent_second_positive_of_valid_coefficients,
    continued_fraction_convergent_second_ne_zero_of_valid_coefficients,
    continued_fraction_numerator_eq_continuant_of_valid_coefficients,
    continued_fraction_value_eq_coefficients_value,
    continued_fraction_continuant_eq_coefficients_continuant,
    continued_fraction_numerator_eq_coefficients_numerator,
    continued_fraction_denominator_eq_coefficients_denominator,
    continued_fraction_convergent_eq_coefficients_convergent,
    continued_fraction_suffix_coefficients_eq_coefficients_suffix,
    continued_fraction_has_valid_suffix_eq_coefficients_valid_suffix,
    continued_fraction_has_positive_suffix_tail_eq_coefficients_suffix_tail,
    continued_fraction_suffix_value_eq_coefficients_suffix_value,
    continued_fraction_suffix_continuant_eq_coefficients_suffix_continuant,
    continued_fraction_suffix_numerator_eq_coefficients_suffix_numerator,
    continued_fraction_suffix_denominator_eq_coefficients_suffix_denominator,
    continued_fraction_suffix_convergent_eq_coefficients_suffix_convergent,
    continued_fraction_suffix_coefficients,
    finite_continued_fraction_suffix_coefficients,
    positive_continued_fraction_suffix_tail, continued_fraction_suffix_value,
    continued_fraction_suffix_continuant, continued_fraction_suffix_numerator,
    continued_fraction_suffix_denominator, continued_fraction_suffix_convergent
from pair import Pair, pair_ext, pair_new_first, pair_new_second

numerals Nat

/// Removing zero coefficients leaves the coefficient list unchanged.
theorem continued_fraction_suffix_coefficients_zero(coefficients: List[Nat]) {
    continued_fraction_suffix_coefficients(coefficients, Nat.0) = coefficients
} by {
    drop_zero(coefficients)
}

/// Every suffix of the empty coefficient list is empty.
theorem continued_fraction_suffix_coefficients_nil(n: Nat) {
    continued_fraction_suffix_coefficients(List.nil[Nat], n) = List.nil[Nat]
} by {
    list_drop_nil[Nat](n)
}

/// Removing a successor number of coefficients from a nonempty coefficient
/// list removes a predecessor number of coefficients from the tail.
theorem continued_fraction_suffix_coefficients_cons_suc(
    head: Nat, tail: List[Nat], n: Nat
) {
    continued_fraction_suffix_coefficients(List.cons(head, tail), n.suc) =
        continued_fraction_suffix_coefficients(tail, n)
} by {
    list_drop_cons_suc(head, tail, n)
}

/// Removing one coefficient from a nonempty coefficient list gives its tail.
theorem continued_fraction_suffix_coefficients_one(head: Nat, tail: List[Nat]) {
    continued_fraction_suffix_coefficients(List.cons(head, tail), Nat.1) = tail
} by {
    drop_one(List.cons(head, tail))
    tail_cancels_cons(head, tail)
}

/// Removing coefficients in two stages is the same as removing their total
/// number.
theorem continued_fraction_suffix_coefficients_twice(
    coefficients: List[Nat], m: Nat, n: Nat
) {
    continued_fraction_suffix_coefficients(
        continued_fraction_suffix_coefficients(coefficients, m), n) =
        continued_fraction_suffix_coefficients(coefficients, m + n)
} by {
    drop_twice(coefficients, m, n)
}

/// Removing all coefficients gives the empty list.
theorem continued_fraction_suffix_coefficients_length(coefficients: List[Nat]) {
    continued_fraction_suffix_coefficients(coefficients, coefficients.length) =
        List.nil[Nat]
} by {
    list_drop_length_nil(coefficients)
}

/// The zero suffix is finite exactly when the original coefficient list is
/// finite.
theorem finite_continued_fraction_suffix_coefficients_zero(
    coefficients: List[Nat]
) {
    finite_continued_fraction_suffix_coefficients(coefficients, Nat.0) =
        finite_continued_fraction_coefficients(coefficients)
} by {
    continued_fraction_suffix_coefficients_zero(coefficients)
}

/// Every suffix of the empty coefficient list is not a finite simple continued
/// fraction.
theorem finite_continued_fraction_suffix_coefficients_nil(n: Nat) {
    not finite_continued_fraction_suffix_coefficients(List.nil[Nat], n)
} by {
    continued_fraction_suffix_coefficients_nil(n)
}

/// Suffix validity over a successor suffix of a nonempty coefficient list is
/// suffix validity over the predecessor suffix of the tail.
theorem finite_continued_fraction_suffix_coefficients_cons_suc(
    head: Nat, tail: List[Nat], n: Nat
) {
    finite_continued_fraction_suffix_coefficients(List.cons(head, tail), n.suc) =
        finite_continued_fraction_suffix_coefficients(tail, n)
} by {
    continued_fraction_suffix_coefficients_cons_suc(head, tail, n)
}

/// The one-step suffix of a nonempty coefficient list is finite exactly when
/// the tail is finite.
theorem finite_continued_fraction_suffix_coefficients_one(
    head: Nat, tail: List[Nat]
) {
    finite_continued_fraction_suffix_coefficients(List.cons(head, tail), Nat.1) =
        finite_continued_fraction_coefficients(tail)
} by {
    continued_fraction_suffix_coefficients_one(head, tail)
}

/// The full-length suffix is not a finite simple continued fraction.
theorem finite_continued_fraction_suffix_coefficients_length(
    coefficients: List[Nat]
) {
    not finite_continued_fraction_suffix_coefficients(
        coefficients, coefficients.length)
} by {
    continued_fraction_suffix_coefficients_length(coefficients)
}

/// The zero suffix has positive tail exactly when the original coefficient
/// list has positive tail.
theorem positive_continued_fraction_suffix_tail_zero(
    coefficients: List[Nat]
) {
    positive_continued_fraction_suffix_tail(coefficients, Nat.0) =
        positive_continued_fraction_tail(coefficients)
} by {
    continued_fraction_suffix_coefficients_zero(coefficients)
}

/// Every suffix of the empty coefficient list has positive tail.
theorem positive_continued_fraction_suffix_tail_nil(n: Nat) {
    positive_continued_fraction_suffix_tail(List.nil[Nat], n)
} by {
    continued_fraction_suffix_coefficients_nil(n)
    positive_continued_fraction_tail_nil
}

/// Suffix-tail positivity over a successor suffix of a nonempty coefficient
/// list is suffix-tail positivity over the predecessor suffix of the tail.
theorem positive_continued_fraction_suffix_tail_cons_suc(
    head: Nat, tail: List[Nat], n: Nat
) {
    positive_continued_fraction_suffix_tail(List.cons(head, tail), n.suc) =
        positive_continued_fraction_suffix_tail(tail, n)
} by {
    continued_fraction_suffix_coefficients_cons_suc(head, tail, n)
}

/// The one-step suffix of a nonempty coefficient list has positive tail
/// exactly when the tail has positive tail.
theorem positive_continued_fraction_suffix_tail_one(
    head: Nat, tail: List[Nat]
) {
    positive_continued_fraction_suffix_tail(List.cons(head, tail), Nat.1) =
        positive_continued_fraction_tail(tail)
} by {
    continued_fraction_suffix_coefficients_one(head, tail)
}

/// The full-length suffix has positive tail.
theorem positive_continued_fraction_suffix_tail_length(
    coefficients: List[Nat]
) {
    positive_continued_fraction_suffix_tail(coefficients, coefficients.length)
} by {
    continued_fraction_suffix_coefficients_length(coefficients)
    positive_continued_fraction_tail_nil
}

/// The zero suffix has the original continued-fraction value.
theorem continued_fraction_suffix_value_zero(coefficients: List[Nat]) {
    continued_fraction_suffix_value(coefficients, Nat.0) =
        continued_fraction_value(coefficients)
} by {
    continued_fraction_suffix_coefficients_zero(coefficients)
}

/// Every suffix of the empty coefficient list has value zero.
theorem continued_fraction_suffix_value_nil(n: Nat) {
    continued_fraction_suffix_value(List.nil[Nat], n) = Rat.0
} by {
    continued_fraction_suffix_coefficients_nil(n)
    continued_fraction_value_nil
}

/// The value of a successor suffix of a nonempty coefficient list is the value
/// of the predecessor suffix of its tail.
theorem continued_fraction_suffix_value_cons_suc(
    head: Nat, tail: List[Nat], n: Nat
) {
    continued_fraction_suffix_value(List.cons(head, tail), n.suc) =
        continued_fraction_suffix_value(tail, n)
} by {
    continued_fraction_suffix_coefficients_cons_suc(head, tail, n)
}

/// The one-step suffix value of a nonempty coefficient list is the value of
/// its tail.
theorem continued_fraction_suffix_value_one(head: Nat, tail: List[Nat]) {
    continued_fraction_suffix_value(List.cons(head, tail), Nat.1) =
        continued_fraction_value(tail)
} by {
    continued_fraction_suffix_coefficients_one(head, tail)
}

/// Suffixing a value in two stages is the same as suffixing by the total
/// number of removed coefficients.
theorem continued_fraction_suffix_value_twice(
    coefficients: List[Nat], m: Nat, n: Nat
) {
    continued_fraction_suffix_value(
        continued_fraction_suffix_coefficients(coefficients, m), n) =
        continued_fraction_suffix_value(coefficients, m + n)
} by {
    continued_fraction_suffix_coefficients_twice(coefficients, m, n)
}

/// The full-length suffix has value zero.
theorem continued_fraction_suffix_value_length(coefficients: List[Nat]) {
    continued_fraction_suffix_value(coefficients, coefficients.length) = Rat.0
} by {
    continued_fraction_suffix_coefficients_length(coefficients)
    continued_fraction_value_nil
}

/// The zero suffix has the original continuant.
theorem continued_fraction_suffix_continuant_zero(coefficients: List[Nat]) {
    continued_fraction_suffix_continuant(coefficients, Nat.0) = continuant(coefficients)
} by {
    continued_fraction_suffix_coefficients_zero(coefficients)
}

/// Every suffix of the empty coefficient list has continuant one.
theorem continued_fraction_suffix_continuant_nil(n: Nat) {
    continued_fraction_suffix_continuant(List.nil[Nat], n) = Nat.1
} by {
    continued_fraction_suffix_coefficients_nil(n)
    continuant_nil
}

/// The continuant of a successor suffix of a nonempty coefficient list is the
/// continuant of the predecessor suffix of its tail.
theorem continued_fraction_suffix_continuant_cons_suc(
    head: Nat, tail: List[Nat], n: Nat
) {
    continued_fraction_suffix_continuant(List.cons(head, tail), n.suc) =
        continued_fraction_suffix_continuant(tail, n)
} by {
    continued_fraction_suffix_coefficients_cons_suc(head, tail, n)
}

/// The one-step suffix continuant of a nonempty coefficient list is the
/// continuant of its tail.
theorem continued_fraction_suffix_continuant_one(head: Nat, tail: List[Nat]) {
    continued_fraction_suffix_continuant(List.cons(head, tail), Nat.1) =
        continuant(tail)
} by {
    continued_fraction_suffix_coefficients_one(head, tail)
}

/// Suffixing a continuant in two stages is the same as suffixing by the total
/// number of removed coefficients.
theorem continued_fraction_suffix_continuant_twice(
    coefficients: List[Nat], m: Nat, n: Nat
) {
    continued_fraction_suffix_continuant(
        continued_fraction_suffix_coefficients(coefficients, m), n) =
        continued_fraction_suffix_continuant(coefficients, m + n)
} by {
    continued_fraction_suffix_coefficients_twice(coefficients, m, n)
}

/// The full-length suffix has continuant one.
theorem continued_fraction_suffix_continuant_length(coefficients: List[Nat]) {
    continued_fraction_suffix_continuant(coefficients, coefficients.length) =
        Nat.1
} by {
    continued_fraction_suffix_coefficients_length(coefficients)
    continuant_nil
}

/// The zero suffix has the original numerator.
theorem continued_fraction_suffix_numerator_zero(coefficients: List[Nat]) {
    continued_fraction_suffix_numerator(coefficients, Nat.0) =
        continued_fraction_numerator(coefficients)
} by {
    continued_fraction_suffix_coefficients_zero(coefficients)
}

/// Every suffix of the empty coefficient list has numerator zero.
theorem continued_fraction_suffix_numerator_nil(n: Nat) {
    continued_fraction_suffix_numerator(List.nil[Nat], n) = Nat.0
} by {
    continued_fraction_suffix_coefficients_nil(n)
    continued_fraction_numerator_nil
}

/// The numerator of a successor suffix of a nonempty coefficient list is the
/// numerator of the predecessor suffix of its tail.
theorem continued_fraction_suffix_numerator_cons_suc(
    head: Nat, tail: List[Nat], n: Nat
) {
    continued_fraction_suffix_numerator(List.cons(head, tail), n.suc) =
        continued_fraction_suffix_numerator(tail, n)
} by {
    continued_fraction_suffix_coefficients_cons_suc(head, tail, n)
}

/// The one-step suffix numerator of a nonempty coefficient list is the
/// numerator of its tail.
theorem continued_fraction_suffix_numerator_one(head: Nat, tail: List[Nat]) {
    continued_fraction_suffix_numerator(List.cons(head, tail), Nat.1) =
        continued_fraction_numerator(tail)
} by {
    continued_fraction_suffix_coefficients_one(head, tail)
}

/// Suffixing a numerator in two stages is the same as suffixing by the total
/// number of removed coefficients.
theorem continued_fraction_suffix_numerator_twice(
    coefficients: List[Nat], m: Nat, n: Nat
) {
    continued_fraction_suffix_numerator(
        continued_fraction_suffix_coefficients(coefficients, m), n) =
        continued_fraction_suffix_numerator(coefficients, m + n)
} by {
    continued_fraction_suffix_coefficients_twice(coefficients, m, n)
}

/// The full-length suffix has numerator zero.
theorem continued_fraction_suffix_numerator_length(coefficients: List[Nat]) {
    continued_fraction_suffix_numerator(coefficients, coefficients.length) =
        Nat.0
} by {
    continued_fraction_suffix_coefficients_length(coefficients)
    continued_fraction_numerator_nil
}

/// The zero suffix has the original denominator.
theorem continued_fraction_suffix_denominator_zero(coefficients: List[Nat]) {
    continued_fraction_suffix_denominator(coefficients, Nat.0) =
        continued_fraction_denominator(coefficients)
} by {
    continued_fraction_suffix_coefficients_zero(coefficients)
}

/// Every suffix of the empty coefficient list has denominator one.
theorem continued_fraction_suffix_denominator_nil(n: Nat) {
    continued_fraction_suffix_denominator(List.nil[Nat], n) = Nat.1
} by {
    continued_fraction_suffix_coefficients_nil(n)
    continued_fraction_denominator_nil
}

/// The denominator of a successor suffix of a nonempty coefficient list is the
/// denominator of the predecessor suffix of its tail.
theorem continued_fraction_suffix_denominator_cons_suc(
    head: Nat, tail: List[Nat], n: Nat
) {
    continued_fraction_suffix_denominator(List.cons(head, tail), n.suc) =
        continued_fraction_suffix_denominator(tail, n)
} by {
    continued_fraction_suffix_coefficients_cons_suc(head, tail, n)
}

/// The one-step suffix denominator of a nonempty coefficient list is the
/// denominator of its tail.
theorem continued_fraction_suffix_denominator_one(head: Nat, tail: List[Nat]) {
    continued_fraction_suffix_denominator(List.cons(head, tail), Nat.1) =
        continued_fraction_denominator(tail)
} by {
    continued_fraction_suffix_coefficients_one(head, tail)
}

/// Suffixing a denominator in two stages is the same as suffixing by the total
/// number of removed coefficients.
theorem continued_fraction_suffix_denominator_twice(
    coefficients: List[Nat], m: Nat, n: Nat
) {
    continued_fraction_suffix_denominator(
        continued_fraction_suffix_coefficients(coefficients, m), n) =
        continued_fraction_suffix_denominator(coefficients, m + n)
} by {
    continued_fraction_suffix_coefficients_twice(coefficients, m, n)
}

/// The full-length suffix has denominator one.
theorem continued_fraction_suffix_denominator_length(coefficients: List[Nat]) {
    continued_fraction_suffix_denominator(coefficients, coefficients.length) =
        Nat.1
} by {
    continued_fraction_suffix_coefficients_length(coefficients)
    continued_fraction_denominator_nil
}

/// The zero suffix has the original convergent.
theorem continued_fraction_suffix_convergent_zero(coefficients: List[Nat]) {
    continued_fraction_suffix_convergent(coefficients, Nat.0) =
        continued_fraction_convergent(coefficients)
} by {
    continued_fraction_suffix_coefficients_zero(coefficients)
}

/// Every suffix of the empty coefficient list has the conventional empty
/// convergent.
theorem continued_fraction_suffix_convergent_nil(n: Nat) {
    continued_fraction_suffix_convergent(List.nil[Nat], n) =
        Pair.new(Nat.0, Nat.1)
} by {
    continued_fraction_suffix_coefficients_nil(n)
    continued_fraction_convergent_nil
}

/// The convergent of a successor suffix of a nonempty coefficient list is the
/// convergent of the predecessor suffix of its tail.
theorem continued_fraction_suffix_convergent_cons_suc(
    head: Nat, tail: List[Nat], n: Nat
) {
    continued_fraction_suffix_convergent(List.cons(head, tail), n.suc) =
        continued_fraction_suffix_convergent(tail, n)
} by {
    continued_fraction_suffix_coefficients_cons_suc(head, tail, n)
}

/// The one-step suffix convergent of a nonempty coefficient list is the
/// convergent of its tail.
theorem continued_fraction_suffix_convergent_one(head: Nat, tail: List[Nat]) {
    continued_fraction_suffix_convergent(List.cons(head, tail), Nat.1) =
        continued_fraction_convergent(tail)
} by {
    continued_fraction_suffix_coefficients_one(head, tail)
}

/// Suffixing a convergent in two stages is the same as suffixing by the total
/// number of removed coefficients.
theorem continued_fraction_suffix_convergent_twice(
    coefficients: List[Nat], m: Nat, n: Nat
) {
    continued_fraction_suffix_convergent(
        continued_fraction_suffix_coefficients(coefficients, m), n) =
        continued_fraction_suffix_convergent(coefficients, m + n)
} by {
    continued_fraction_suffix_coefficients_twice(coefficients, m, n)
}

/// The full-length suffix has the conventional empty convergent.
theorem continued_fraction_suffix_convergent_length(coefficients: List[Nat]) {
    continued_fraction_suffix_convergent(coefficients, coefficients.length) =
        Pair.new(Nat.0, Nat.1)
} by {
    continued_fraction_suffix_coefficients_length(coefficients)
    continued_fraction_convergent_nil
}

/// The convergent numerator of a suffix is its suffix numerator.
theorem continued_fraction_suffix_convergent_first_eq_numerator(
    coefficients: List[Nat], n: Nat
) {
    continued_fraction_suffix_convergent(coefficients, n).first =
        continued_fraction_suffix_numerator(coefficients, n)
} by {
    continued_fraction_convergent_first_eq_numerator(
        continued_fraction_suffix_coefficients(coefficients, n))
}

/// The suffix numerator is the convergent numerator of the suffix.
theorem continued_fraction_suffix_numerator_eq_convergent_first(
    coefficients: List[Nat], n: Nat
) {
    continued_fraction_suffix_numerator(coefficients, n) =
        continued_fraction_suffix_convergent(coefficients, n).first
} by {
    continued_fraction_suffix_convergent_first_eq_numerator(coefficients, n)
}

/// The convergent denominator of a suffix is its suffix denominator.
theorem continued_fraction_suffix_convergent_second_eq_denominator(
    coefficients: List[Nat], n: Nat
) {
    continued_fraction_suffix_convergent(coefficients, n).second =
        continued_fraction_suffix_denominator(coefficients, n)
} by {
    continued_fraction_convergent_second_eq_denominator(
        continued_fraction_suffix_coefficients(coefficients, n))
}

/// The suffix denominator is the convergent denominator of the suffix.
theorem continued_fraction_suffix_denominator_eq_convergent_second(
    coefficients: List[Nat], n: Nat
) {
    continued_fraction_suffix_denominator(coefficients, n) =
        continued_fraction_suffix_convergent(coefficients, n).second
} by {
    continued_fraction_suffix_convergent_second_eq_denominator(coefficients, n)
}

/// The suffix convergent is the pair of the suffix numerator and denominator.
theorem continued_fraction_suffix_convergent_eq_numerator_denominator_pair(
    coefficients: List[Nat], n: Nat
) {
    continued_fraction_suffix_convergent(coefficients, n) =
        Pair.new(continued_fraction_suffix_numerator(coefficients, n),
            continued_fraction_suffix_denominator(coefficients, n))
} by {
    continued_fraction_suffix_convergent_first_eq_numerator(coefficients, n)
    continued_fraction_suffix_convergent_second_eq_denominator(coefficients, n)
    pair_new_first(continued_fraction_suffix_numerator(coefficients, n),
        continued_fraction_suffix_denominator(coefficients, n))
    pair_new_second(continued_fraction_suffix_numerator(coefficients, n),
        continued_fraction_suffix_denominator(coefficients, n))
    pair_ext(continued_fraction_suffix_convergent(coefficients, n),
        Pair.new(continued_fraction_suffix_numerator(coefficients, n),
            continued_fraction_suffix_denominator(coefficients, n)))
}

/// A valid suffix has positive denominator.
theorem continued_fraction_suffix_denominator_positive_of_valid_coefficients(
    coefficients: List[Nat], n: Nat
) {
    finite_continued_fraction_suffix_coefficients(coefficients, n)
        implies Nat.0 < continued_fraction_suffix_denominator(coefficients, n)
} by {
    if finite_continued_fraction_suffix_coefficients(coefficients, n) {
        continued_fraction_denominator_positive_of_valid_coefficients(
            continued_fraction_suffix_coefficients(coefficients, n))
        Nat.0 < continued_fraction_denominator(
            continued_fraction_suffix_coefficients(coefficients, n))
        Nat.0 < continued_fraction_suffix_denominator(coefficients, n)
    }
}

/// A valid suffix has nonzero denominator.
theorem continued_fraction_suffix_denominator_ne_zero_of_valid_coefficients(
    coefficients: List[Nat], n: Nat
) {
    finite_continued_fraction_suffix_coefficients(coefficients, n)
        implies continued_fraction_suffix_denominator(coefficients, n) != Nat.0
} by {
    if finite_continued_fraction_suffix_coefficients(coefficients, n) {
        continued_fraction_denominator_ne_zero_of_valid_coefficients(
            continued_fraction_suffix_coefficients(coefficients, n))
        continued_fraction_suffix_denominator(coefficients, n) != Nat.0
    }
}

/// A valid suffix has positive convergent denominator.
theorem continued_fraction_suffix_convergent_second_positive_of_valid_coefficients(
    coefficients: List[Nat], n: Nat
) {
    finite_continued_fraction_suffix_coefficients(coefficients, n)
        implies Nat.0 < continued_fraction_suffix_convergent(coefficients, n).second
} by {
    if finite_continued_fraction_suffix_coefficients(coefficients, n) {
        continued_fraction_convergent_second_positive_of_valid_coefficients(
            continued_fraction_suffix_coefficients(coefficients, n))
        Nat.0 < continued_fraction_suffix_convergent(coefficients, n).second
    }
}

/// A valid suffix has nonzero convergent denominator.
theorem continued_fraction_suffix_convergent_second_ne_zero_of_valid_coefficients(
    coefficients: List[Nat], n: Nat
) {
    finite_continued_fraction_suffix_coefficients(coefficients, n)
        implies continued_fraction_suffix_convergent(coefficients, n).second != Nat.0
} by {
    if finite_continued_fraction_suffix_coefficients(coefficients, n) {
        continued_fraction_convergent_second_ne_zero_of_valid_coefficients(
            continued_fraction_suffix_coefficients(coefficients, n))
        continued_fraction_suffix_convergent(coefficients, n).second != Nat.0
    }
}

/// A valid suffix has numerator equal to its continuant.
theorem continued_fraction_suffix_numerator_eq_continuant_of_valid_coefficients(
    coefficients: List[Nat], n: Nat
) {
    finite_continued_fraction_suffix_coefficients(coefficients, n)
        implies continued_fraction_suffix_numerator(coefficients, n) =
            continued_fraction_suffix_continuant(coefficients, n)
} by {
    if finite_continued_fraction_suffix_coefficients(coefficients, n) {
        continued_fraction_numerator_eq_continuant_of_valid_coefficients(
            continued_fraction_suffix_coefficients(coefficients, n))
        continued_fraction_numerator(continued_fraction_suffix_coefficients(coefficients, n)) =
            continuant(continued_fraction_suffix_coefficients(coefficients, n))
        continued_fraction_suffix_numerator(coefficients, n) =
            continued_fraction_suffix_continuant(coefficients, n)
    }
}

/// A valid suffix has convergent numerator equal to its continuant.
theorem continued_fraction_suffix_convergent_first_eq_continuant_of_valid_coefficients(
    coefficients: List[Nat], n: Nat
) {
    finite_continued_fraction_suffix_coefficients(coefficients, n)
        implies continued_fraction_suffix_convergent(coefficients, n).first =
            continued_fraction_suffix_continuant(coefficients, n)
} by {
    if finite_continued_fraction_suffix_coefficients(coefficients, n) {
        continued_fraction_suffix_convergent_first_eq_numerator(coefficients, n)
        continued_fraction_suffix_numerator_eq_continuant_of_valid_coefficients(
            coefficients, n)
    }
}

/// A nonempty coefficient list unfolds through its one-step suffix.
theorem continued_fraction_value_cons_suffix_one(head: Nat, tail: List[Nat]) {
    continued_fraction_value(List.cons(head, tail)) =
        Rat.from_nat(head) +
            continued_fraction_suffix_value(List.cons(head, tail), Nat.1).inverse
} by {
    continued_fraction_value_cons(head, tail)
    continued_fraction_suffix_value_one(head, tail)
}

/// The suffix-coefficients method has the zero-suffix coefficient list.
theorem continued_fraction_suffix_coefficients_zero_method(cf: ContinuedFraction) {
    cf.suffix_coefficients(Nat.0) = cf.coefficients
} by {
    continued_fraction_suffix_coefficients_eq_coefficients_suffix(cf, Nat.0)
    continued_fraction_suffix_coefficients_zero(cf.coefficients)
}

/// The zero suffix of a continued fraction is valid.
theorem continued_fraction_has_valid_suffix_zero(cf: ContinuedFraction) {
    cf.has_valid_suffix(Nat.0)
} by {
    continued_fraction_has_valid_suffix_eq_coefficients_valid_suffix(cf, Nat.0)
    finite_continued_fraction_suffix_coefficients_zero(cf.coefficients)
    continued_fraction_coefficients_valid(cf)
}

/// The zero suffix has positive tail exactly when the whole coefficient list
/// has positive tail.
theorem continued_fraction_has_positive_suffix_tail_zero_eq(
    cf: ContinuedFraction
) {
    cf.has_positive_suffix_tail(Nat.0) =
        positive_continued_fraction_tail(cf.coefficients)
} by {
    continued_fraction_has_positive_suffix_tail_eq_coefficients_suffix_tail(
        cf, Nat.0)
    positive_continued_fraction_suffix_tail_zero(cf.coefficients)
}

/// The zero suffix has the original value.
theorem continued_fraction_suffix_value_zero_method(cf: ContinuedFraction) {
    cf.suffix_value(Nat.0) = cf.value
} by {
    continued_fraction_suffix_value_eq_coefficients_suffix_value(cf, Nat.0)
    continued_fraction_suffix_value_zero(cf.coefficients)
    continued_fraction_value_eq_coefficients_value(cf)
}

/// The zero suffix has the original continuant.
theorem continued_fraction_suffix_continuant_zero_method(cf: ContinuedFraction) {
    cf.suffix_continuant(Nat.0) = cf.continuant
} by {
    continued_fraction_suffix_continuant_eq_coefficients_suffix_continuant(cf, Nat.0)
    continued_fraction_suffix_continuant_zero(cf.coefficients)
    continued_fraction_continuant_eq_coefficients_continuant(cf)
}

/// The zero suffix has the original numerator.
theorem continued_fraction_suffix_numerator_zero_method(cf: ContinuedFraction) {
    cf.suffix_numerator(Nat.0) = cf.numerator
} by {
    continued_fraction_suffix_numerator_eq_coefficients_suffix_numerator(cf, Nat.0)
    continued_fraction_suffix_numerator_zero(cf.coefficients)
    continued_fraction_numerator_eq_coefficients_numerator(cf)
}

/// The zero suffix has the original denominator.
theorem continued_fraction_suffix_denominator_zero_method(cf: ContinuedFraction) {
    cf.suffix_denominator(Nat.0) = cf.denominator
} by {
    continued_fraction_suffix_denominator_eq_coefficients_suffix_denominator(
        cf, Nat.0)
    continued_fraction_suffix_denominator_zero(cf.coefficients)
    continued_fraction_denominator_eq_coefficients_denominator(cf)
}

/// The zero suffix has the original convergent.
theorem continued_fraction_suffix_convergent_zero_method(cf: ContinuedFraction) {
    cf.suffix_convergent(Nat.0) = cf.convergent
} by {
    continued_fraction_suffix_convergent_eq_coefficients_suffix_convergent(
        cf, Nat.0)
    continued_fraction_suffix_convergent_zero(cf.coefficients)
    continued_fraction_convergent_eq_coefficients_convergent(cf)
}

/// The full-length suffix of a continued fraction is empty.
theorem continued_fraction_suffix_coefficients_length_method(cf: ContinuedFraction) {
    cf.suffix_coefficients(cf.coefficients.length) = List.nil[Nat]
} by {
    continued_fraction_suffix_coefficients_eq_coefficients_suffix(
        cf, cf.coefficients.length)
    continued_fraction_suffix_coefficients_length(cf.coefficients)
}

/// The full-length suffix of a continued fraction is not valid.
theorem continued_fraction_has_valid_suffix_length_false(cf: ContinuedFraction) {
    not cf.has_valid_suffix(cf.coefficients.length)
} by {
    continued_fraction_has_valid_suffix_eq_coefficients_valid_suffix(
        cf, cf.coefficients.length)
    finite_continued_fraction_suffix_coefficients_length(cf.coefficients)
}

/// The full-length suffix of a continued fraction has positive tail.
theorem continued_fraction_has_positive_suffix_tail_length(cf: ContinuedFraction) {
    cf.has_positive_suffix_tail(cf.coefficients.length)
} by {
    continued_fraction_has_positive_suffix_tail_eq_coefficients_suffix_tail(
        cf, cf.coefficients.length)
    positive_continued_fraction_suffix_tail_length(cf.coefficients)
}

/// The full-length suffix of a continued fraction has value zero.
theorem continued_fraction_suffix_value_length_method(cf: ContinuedFraction) {
    cf.suffix_value(cf.coefficients.length) = Rat.0
} by {
    continued_fraction_suffix_value_eq_coefficients_suffix_value(
        cf, cf.coefficients.length)
    continued_fraction_suffix_value_length(cf.coefficients)
}

/// The full-length suffix of a continued fraction has continuant one.
theorem continued_fraction_suffix_continuant_length_method(cf: ContinuedFraction) {
    cf.suffix_continuant(cf.coefficients.length) = Nat.1
} by {
    continued_fraction_suffix_continuant_eq_coefficients_suffix_continuant(
        cf, cf.coefficients.length)
    continued_fraction_suffix_continuant_length(cf.coefficients)
}

/// The full-length suffix of a continued fraction has numerator zero.
theorem continued_fraction_suffix_numerator_length_method(cf: ContinuedFraction) {
    cf.suffix_numerator(cf.coefficients.length) = Nat.0
} by {
    continued_fraction_suffix_numerator_eq_coefficients_suffix_numerator(
        cf, cf.coefficients.length)
    continued_fraction_suffix_numerator_length(cf.coefficients)
}

/// The full-length suffix of a continued fraction has denominator one.
theorem continued_fraction_suffix_denominator_length_method(cf: ContinuedFraction) {
    cf.suffix_denominator(cf.coefficients.length) = Nat.1
} by {
    continued_fraction_suffix_denominator_eq_coefficients_suffix_denominator(
        cf, cf.coefficients.length)
    continued_fraction_suffix_denominator_length(cf.coefficients)
}

/// The full-length suffix of a continued fraction has the conventional empty
/// convergent.
theorem continued_fraction_suffix_convergent_length_method(cf: ContinuedFraction) {
    cf.suffix_convergent(cf.coefficients.length) = Pair.new(Nat.0, Nat.1)
} by {
    continued_fraction_suffix_convergent_eq_coefficients_suffix_convergent(
        cf, cf.coefficients.length)
    continued_fraction_suffix_convergent_length(cf.coefficients)
}

/// A displayed nonempty continued fraction has one-step suffix coefficients
/// equal to the displayed tail.
theorem continued_fraction_suffix_coefficients_one_method(
    cf: ContinuedFraction, head: Nat, tail: List[Nat]
) {
    cf.coefficients = List.cons(head, tail)
        implies cf.suffix_coefficients(Nat.1) = tail
} by {
    if cf.coefficients = List.cons(head, tail) {
        continued_fraction_suffix_coefficients_eq_coefficients_suffix(cf, Nat.1)
        continued_fraction_suffix_coefficients_one(head, tail)
    }
}

/// A displayed nonempty continued fraction has one-step suffix value equal to
/// the value of the displayed tail.
theorem continued_fraction_suffix_value_one_method(
    cf: ContinuedFraction, head: Nat, tail: List[Nat]
) {
    cf.coefficients = List.cons(head, tail)
        implies cf.suffix_value(Nat.1) = continued_fraction_value(tail)
} by {
    if cf.coefficients = List.cons(head, tail) {
        continued_fraction_suffix_value_eq_coefficients_suffix_value(cf, Nat.1)
        continued_fraction_suffix_value_one(head, tail)
    }
}

/// A displayed nonempty continued fraction has one-step suffix continuant
/// equal to the continuant of the displayed tail.
theorem continued_fraction_suffix_continuant_one_method(
    cf: ContinuedFraction, head: Nat, tail: List[Nat]
) {
    cf.coefficients = List.cons(head, tail)
        implies cf.suffix_continuant(Nat.1) = continuant(tail)
} by {
    if cf.coefficients = List.cons(head, tail) {
        continued_fraction_suffix_continuant_eq_coefficients_suffix_continuant(
            cf, Nat.1)
        continued_fraction_suffix_continuant_one(head, tail)
    }
}

/// A displayed nonempty continued fraction has one-step suffix numerator equal
/// to the numerator of the displayed tail.
theorem continued_fraction_suffix_numerator_one_method(
    cf: ContinuedFraction, head: Nat, tail: List[Nat]
) {
    cf.coefficients = List.cons(head, tail)
        implies cf.suffix_numerator(Nat.1) = continued_fraction_numerator(tail)
} by {
    if cf.coefficients = List.cons(head, tail) {
        continued_fraction_suffix_numerator_eq_coefficients_suffix_numerator(
            cf, Nat.1)
        continued_fraction_suffix_numerator_one(head, tail)
    }
}

/// A displayed nonempty continued fraction has one-step suffix denominator
/// equal to the denominator of the displayed tail.
theorem continued_fraction_suffix_denominator_one_method(
    cf: ContinuedFraction, head: Nat, tail: List[Nat]
) {
    cf.coefficients = List.cons(head, tail)
        implies cf.suffix_denominator(Nat.1) = continued_fraction_denominator(tail)
} by {
    if cf.coefficients = List.cons(head, tail) {
        continued_fraction_suffix_denominator_eq_coefficients_suffix_denominator(
            cf, Nat.1)
        continued_fraction_suffix_denominator_one(head, tail)
    }
}

/// A displayed nonempty continued fraction has one-step suffix convergent
/// equal to the convergent of the displayed tail.
theorem continued_fraction_suffix_convergent_one_method(
    cf: ContinuedFraction, head: Nat, tail: List[Nat]
) {
    cf.coefficients = List.cons(head, tail)
        implies cf.suffix_convergent(Nat.1) = continued_fraction_convergent(tail)
} by {
    if cf.coefficients = List.cons(head, tail) {
        continued_fraction_suffix_convergent_eq_coefficients_suffix_convergent(
            cf, Nat.1)
        continued_fraction_suffix_convergent_one(head, tail)
    }
}

/// A displayed nonempty continued fraction unfolds through its one-step
/// suffix value.
theorem continued_fraction_value_cons_suffix_one_method(
    cf: ContinuedFraction, head: Nat, tail: List[Nat]
) {
    cf.coefficients = List.cons(head, tail)
        implies cf.value =
            Rat.from_nat(head) + cf.suffix_value(Nat.1).inverse
} by {
    if cf.coefficients = List.cons(head, tail) {
        continued_fraction_value_eq_coefficients_value(cf)
        continued_fraction_value_cons_suffix_one(head, tail)
        continued_fraction_suffix_value_one_method(cf, head, tail)
    }
}

/// A valid suffix of a continued fraction has positive denominator.
theorem continued_fraction_suffix_denominator_positive(
    cf: ContinuedFraction, n: Nat
) {
    cf.has_valid_suffix(n) implies Nat.0 < cf.suffix_denominator(n)
} by {
    if cf.has_valid_suffix(n) {
        continued_fraction_has_valid_suffix_eq_coefficients_valid_suffix(cf, n)
        continued_fraction_suffix_denominator_positive_of_valid_coefficients(
            cf.coefficients, n)
        continued_fraction_suffix_denominator_eq_coefficients_suffix_denominator(
            cf, n)
        Nat.0 < cf.suffix_denominator(n)
    }
}

/// A valid suffix of a continued fraction has nonzero denominator.
theorem continued_fraction_suffix_denominator_ne_zero(
    cf: ContinuedFraction, n: Nat
) {
    cf.has_valid_suffix(n) implies cf.suffix_denominator(n) != Nat.0
} by {
    if cf.has_valid_suffix(n) {
        continued_fraction_suffix_denominator_positive(cf, n)
        cf.suffix_denominator(n) != Nat.0
    }
}

/// A valid suffix of a continued fraction has positive convergent denominator.
theorem continued_fraction_suffix_convergent_second_positive(
    cf: ContinuedFraction, n: Nat
) {
    cf.has_valid_suffix(n) implies Nat.0 < cf.suffix_convergent(n).second
} by {
    if cf.has_valid_suffix(n) {
        continued_fraction_has_valid_suffix_eq_coefficients_valid_suffix(cf, n)
        continued_fraction_suffix_convergent_second_positive_of_valid_coefficients(
            cf.coefficients, n)
        continued_fraction_suffix_convergent_eq_coefficients_suffix_convergent(
            cf, n)
        Nat.0 < cf.suffix_convergent(n).second
    }
}

/// A valid suffix of a continued fraction has nonzero convergent denominator.
theorem continued_fraction_suffix_convergent_second_ne_zero(
    cf: ContinuedFraction, n: Nat
) {
    cf.has_valid_suffix(n) implies cf.suffix_convergent(n).second != Nat.0
} by {
    if cf.has_valid_suffix(n) {
        continued_fraction_suffix_convergent_second_positive(cf, n)
        cf.suffix_convergent(n).second != Nat.0
    }
}

/// A displayed nonempty continued fraction has successor suffix coefficients
/// equal to the predecessor suffix coefficients of the displayed tail.
theorem continued_fraction_suffix_coefficients_cons_suc_method(
    cf: ContinuedFraction, head: Nat, tail: List[Nat], n: Nat
) {
    cf.coefficients = List.cons(head, tail)
        implies cf.suffix_coefficients(n.suc) =
            continued_fraction_suffix_coefficients(tail, n)
} by {
    if cf.coefficients = List.cons(head, tail) {
        continued_fraction_suffix_coefficients_eq_coefficients_suffix(cf, n.suc)
        continued_fraction_suffix_coefficients_cons_suc(head, tail, n)
    }
}

/// A displayed nonempty continued fraction has successor suffix value equal to
/// the predecessor suffix value of the displayed tail.
theorem continued_fraction_suffix_value_cons_suc_method(
    cf: ContinuedFraction, head: Nat, tail: List[Nat], n: Nat
) {
    cf.coefficients = List.cons(head, tail)
        implies cf.suffix_value(n.suc) = continued_fraction_suffix_value(tail, n)
} by {
    if cf.coefficients = List.cons(head, tail) {
        continued_fraction_suffix_value_eq_coefficients_suffix_value(cf, n.suc)
        continued_fraction_suffix_value_cons_suc(head, tail, n)
    }
}

/// A displayed nonempty continued fraction has successor suffix continuant
/// equal to the predecessor suffix continuant of the displayed tail.
theorem continued_fraction_suffix_continuant_cons_suc_method(
    cf: ContinuedFraction, head: Nat, tail: List[Nat], n: Nat
) {
    cf.coefficients = List.cons(head, tail)
        implies cf.suffix_continuant(n.suc) =
            continued_fraction_suffix_continuant(tail, n)
} by {
    if cf.coefficients = List.cons(head, tail) {
        continued_fraction_suffix_continuant_eq_coefficients_suffix_continuant(
            cf, n.suc)
        continued_fraction_suffix_continuant_cons_suc(head, tail, n)
    }
}

/// A displayed nonempty continued fraction has successor suffix numerator
/// equal to the predecessor suffix numerator of the displayed tail.
theorem continued_fraction_suffix_numerator_cons_suc_method(
    cf: ContinuedFraction, head: Nat, tail: List[Nat], n: Nat
) {
    cf.coefficients = List.cons(head, tail)
        implies cf.suffix_numerator(n.suc) =
            continued_fraction_suffix_numerator(tail, n)
} by {
    if cf.coefficients = List.cons(head, tail) {
        continued_fraction_suffix_numerator_eq_coefficients_suffix_numerator(
            cf, n.suc)
        continued_fraction_suffix_numerator_cons_suc(head, tail, n)
    }
}

/// A displayed nonempty continued fraction has successor suffix denominator
/// equal to the predecessor suffix denominator of the displayed tail.
theorem continued_fraction_suffix_denominator_cons_suc_method(
    cf: ContinuedFraction, head: Nat, tail: List[Nat], n: Nat
) {
    cf.coefficients = List.cons(head, tail)
        implies cf.suffix_denominator(n.suc) =
            continued_fraction_suffix_denominator(tail, n)
} by {
    if cf.coefficients = List.cons(head, tail) {
        continued_fraction_suffix_denominator_eq_coefficients_suffix_denominator(
            cf, n.suc)
        continued_fraction_suffix_denominator_cons_suc(head, tail, n)
    }
}

/// A displayed nonempty continued fraction has successor suffix convergent
/// equal to the predecessor suffix convergent of the displayed tail.
theorem continued_fraction_suffix_convergent_cons_suc_method(
    cf: ContinuedFraction, head: Nat, tail: List[Nat], n: Nat
) {
    cf.coefficients = List.cons(head, tail)
        implies cf.suffix_convergent(n.suc) =
            continued_fraction_suffix_convergent(tail, n)
} by {
    if cf.coefficients = List.cons(head, tail) {
        continued_fraction_suffix_convergent_eq_coefficients_suffix_convergent(
            cf, n.suc)
        continued_fraction_suffix_convergent_cons_suc(head, tail, n)
    }
}

/// Removing coefficients from a continued fraction in two stages is the same
/// as removing their total number.
theorem continued_fraction_suffix_coefficients_twice_method(
    cf: ContinuedFraction, m: Nat, n: Nat
) {
    continued_fraction_suffix_coefficients(cf.suffix_coefficients(m), n) =
        cf.suffix_coefficients(m + n)
} by {
    continued_fraction_suffix_coefficients_eq_coefficients_suffix(cf, m)
    continued_fraction_suffix_coefficients_eq_coefficients_suffix(cf, m + n)
    continued_fraction_suffix_coefficients_twice(cf.coefficients, m, n)
}

/// The value of a two-stage suffix of a continued fraction is the value of the
/// total suffix.
theorem continued_fraction_suffix_value_twice_method(
    cf: ContinuedFraction, m: Nat, n: Nat
) {
    continued_fraction_suffix_value(cf.suffix_coefficients(m), n) =
        cf.suffix_value(m + n)
} by {
    continued_fraction_suffix_coefficients_eq_coefficients_suffix(cf, m)
    continued_fraction_suffix_value_eq_coefficients_suffix_value(cf, m + n)
    continued_fraction_suffix_value_twice(cf.coefficients, m, n)
}

/// The continuant of a two-stage suffix of a continued fraction is the
/// continuant of the total suffix.
theorem continued_fraction_suffix_continuant_twice_method(
    cf: ContinuedFraction, m: Nat, n: Nat
) {
    continued_fraction_suffix_continuant(cf.suffix_coefficients(m), n) =
        cf.suffix_continuant(m + n)
} by {
    continued_fraction_suffix_coefficients_eq_coefficients_suffix(cf, m)
    continued_fraction_suffix_continuant_eq_coefficients_suffix_continuant(
        cf, m + n)
    continued_fraction_suffix_continuant_twice(cf.coefficients, m, n)
}

/// The numerator of a two-stage suffix of a continued fraction is the
/// numerator of the total suffix.
theorem continued_fraction_suffix_numerator_twice_method(
    cf: ContinuedFraction, m: Nat, n: Nat
) {
    continued_fraction_suffix_numerator(cf.suffix_coefficients(m), n) =
        cf.suffix_numerator(m + n)
} by {
    continued_fraction_suffix_coefficients_eq_coefficients_suffix(cf, m)
    continued_fraction_suffix_numerator_eq_coefficients_suffix_numerator(
        cf, m + n)
    continued_fraction_suffix_numerator_twice(cf.coefficients, m, n)
}

/// The denominator of a two-stage suffix of a continued fraction is the
/// denominator of the total suffix.
theorem continued_fraction_suffix_denominator_twice_method(
    cf: ContinuedFraction, m: Nat, n: Nat
) {
    continued_fraction_suffix_denominator(cf.suffix_coefficients(m), n) =
        cf.suffix_denominator(m + n)
} by {
    continued_fraction_suffix_coefficients_eq_coefficients_suffix(cf, m)
    continued_fraction_suffix_denominator_eq_coefficients_suffix_denominator(
        cf, m + n)
    continued_fraction_suffix_denominator_twice(cf.coefficients, m, n)
}

/// The convergent of a two-stage suffix of a continued fraction is the
/// convergent of the total suffix.
theorem continued_fraction_suffix_convergent_twice_method(
    cf: ContinuedFraction, m: Nat, n: Nat
) {
    continued_fraction_suffix_convergent(cf.suffix_coefficients(m), n) =
        cf.suffix_convergent(m + n)
} by {
    continued_fraction_suffix_coefficients_eq_coefficients_suffix(cf, m)
    continued_fraction_suffix_convergent_eq_coefficients_suffix_convergent(
        cf, m + n)
    continued_fraction_suffix_convergent_twice(cf.coefficients, m, n)
}

/// The suffix-validity method implies coefficient-list suffix validity.
theorem continued_fraction_has_valid_suffix_elim(
    cf: ContinuedFraction, n: Nat
) {
    cf.has_valid_suffix(n)
        implies finite_continued_fraction_suffix_coefficients(cf.coefficients, n)
} by {
    if cf.has_valid_suffix(n) {
        continued_fraction_has_valid_suffix_eq_coefficients_valid_suffix(cf, n)
        finite_continued_fraction_suffix_coefficients(cf.coefficients, n)
    }
}

/// Coefficient-list suffix validity implies the suffix-validity method.
theorem continued_fraction_has_valid_suffix_intro(
    cf: ContinuedFraction, n: Nat
) {
    finite_continued_fraction_suffix_coefficients(cf.coefficients, n)
        implies cf.has_valid_suffix(n)
} by {
    if finite_continued_fraction_suffix_coefficients(cf.coefficients, n) {
        continued_fraction_has_valid_suffix_eq_coefficients_valid_suffix(cf, n)
        cf.has_valid_suffix(n)
    }
}

/// The suffix-tail method implies coefficient-list suffix-tail positivity.
theorem continued_fraction_has_positive_suffix_tail_elim(
    cf: ContinuedFraction, n: Nat
) {
    cf.has_positive_suffix_tail(n)
        implies positive_continued_fraction_suffix_tail(cf.coefficients, n)
} by {
    if cf.has_positive_suffix_tail(n) {
        continued_fraction_has_positive_suffix_tail_eq_coefficients_suffix_tail(
            cf, n)
        positive_continued_fraction_suffix_tail(cf.coefficients, n)
    }
}

/// Coefficient-list suffix-tail positivity implies the suffix-tail method.
theorem continued_fraction_has_positive_suffix_tail_intro(
    cf: ContinuedFraction, n: Nat
) {
    positive_continued_fraction_suffix_tail(cf.coefficients, n)
        implies cf.has_positive_suffix_tail(n)
} by {
    if positive_continued_fraction_suffix_tail(cf.coefficients, n) {
        continued_fraction_has_positive_suffix_tail_eq_coefficients_suffix_tail(
            cf, n)
        cf.has_positive_suffix_tail(n)
    }
}

/// The suffix convergent numerator method is the suffix numerator method.
theorem continued_fraction_suffix_convergent_first_eq_suffix_numerator(
    cf: ContinuedFraction, n: Nat
) {
    cf.suffix_convergent(n).first = cf.suffix_numerator(n)
} by {
    continued_fraction_suffix_convergent_eq_coefficients_suffix_convergent(cf, n)
    continued_fraction_suffix_numerator_eq_coefficients_suffix_numerator(cf, n)
    continued_fraction_suffix_convergent_first_eq_numerator(cf.coefficients, n)
}

/// The suffix numerator method is the suffix convergent numerator method.
theorem continued_fraction_suffix_numerator_eq_suffix_convergent_first(
    cf: ContinuedFraction, n: Nat
) {
    cf.suffix_numerator(n) = cf.suffix_convergent(n).first
} by {
    continued_fraction_suffix_convergent_first_eq_suffix_numerator(cf, n)
}

/// The suffix convergent denominator method is the suffix denominator method.
theorem continued_fraction_suffix_convergent_second_eq_suffix_denominator(
    cf: ContinuedFraction, n: Nat
) {
    cf.suffix_convergent(n).second = cf.suffix_denominator(n)
} by {
    continued_fraction_suffix_convergent_eq_coefficients_suffix_convergent(cf, n)
    continued_fraction_suffix_denominator_eq_coefficients_suffix_denominator(cf, n)
    continued_fraction_suffix_convergent_second_eq_denominator(cf.coefficients, n)
}

/// The suffix denominator method is the suffix convergent denominator method.
theorem continued_fraction_suffix_denominator_eq_suffix_convergent_second(
    cf: ContinuedFraction, n: Nat
) {
    cf.suffix_denominator(n) = cf.suffix_convergent(n).second
} by {
    continued_fraction_suffix_convergent_second_eq_suffix_denominator(cf, n)
}

/// A valid suffix has numerator method equal to continuant method.
theorem continued_fraction_suffix_numerator_eq_suffix_continuant_of_valid_suffix(
    cf: ContinuedFraction, n: Nat
) {
    cf.has_valid_suffix(n)
        implies cf.suffix_numerator(n) = cf.suffix_continuant(n)
} by {
    if cf.has_valid_suffix(n) {
        continued_fraction_has_valid_suffix_elim(cf, n)
        continued_fraction_suffix_numerator_eq_continuant_of_valid_coefficients(
            cf.coefficients, n)
        continued_fraction_suffix_numerator_eq_coefficients_suffix_numerator(cf, n)
        continued_fraction_suffix_continuant_eq_coefficients_suffix_continuant(cf, n)
        cf.suffix_numerator(n) = cf.suffix_continuant(n)
    }
}

/// A valid suffix has convergent numerator method equal to continuant method.
theorem continued_fraction_suffix_convergent_first_eq_suffix_continuant_of_valid_suffix(
    cf: ContinuedFraction, n: Nat
) {
    cf.has_valid_suffix(n)
        implies cf.suffix_convergent(n).first = cf.suffix_continuant(n)
} by {
    if cf.has_valid_suffix(n) {
        continued_fraction_suffix_convergent_first_eq_suffix_numerator(cf, n)
        continued_fraction_suffix_numerator_eq_suffix_continuant_of_valid_suffix(
            cf, n)
    }
}

/// The suffix convergent method is the pair of the suffix numerator and
/// denominator methods.
theorem continued_fraction_suffix_convergent_eq_suffix_numerator_denominator_pair(
    cf: ContinuedFraction, n: Nat
) {
    cf.suffix_convergent(n) =
        Pair.new(cf.suffix_numerator(n), cf.suffix_denominator(n))
} by {
    continued_fraction_suffix_convergent_eq_coefficients_suffix_convergent(cf, n)
    continued_fraction_suffix_numerator_eq_coefficients_suffix_numerator(cf, n)
    continued_fraction_suffix_denominator_eq_coefficients_suffix_denominator(cf, n)
    continued_fraction_suffix_convergent_eq_numerator_denominator_pair(
        cf.coefficients, n)
}

/// A valid suffix has continuant equal to its numerator.
theorem continued_fraction_suffix_continuant_eq_numerator_of_valid_coefficients(
    coefficients: List[Nat], n: Nat
) {
    finite_continued_fraction_suffix_coefficients(coefficients, n)
        implies continued_fraction_suffix_continuant(coefficients, n) =
            continued_fraction_suffix_numerator(coefficients, n)
} by {
    if finite_continued_fraction_suffix_coefficients(coefficients, n) {
        continued_fraction_suffix_numerator_eq_continuant_of_valid_coefficients(
            coefficients, n)
    }
}

/// A valid suffix has continuant method equal to numerator method.
theorem continued_fraction_suffix_continuant_eq_suffix_numerator_of_valid_suffix(
    cf: ContinuedFraction, n: Nat
) {
    cf.has_valid_suffix(n)
        implies cf.suffix_continuant(n) = cf.suffix_numerator(n)
} by {
    if cf.has_valid_suffix(n) {
        continued_fraction_suffix_numerator_eq_suffix_continuant_of_valid_suffix(
            cf, n)
    }
}

/// A valid suffix has convergent pair given by its continuant and denominator.
theorem continued_fraction_suffix_convergent_eq_continuant_denominator_pair_of_valid_coefficients(
    coefficients: List[Nat], n: Nat
) {
    finite_continued_fraction_suffix_coefficients(coefficients, n)
        implies continued_fraction_suffix_convergent(coefficients, n) =
            Pair.new(continued_fraction_suffix_continuant(coefficients, n),
                continued_fraction_suffix_denominator(coefficients, n))
} by {
    if finite_continued_fraction_suffix_coefficients(coefficients, n) {
        continued_fraction_suffix_convergent_eq_numerator_denominator_pair(
            coefficients, n)
        continued_fraction_suffix_numerator_eq_continuant_of_valid_coefficients(
            coefficients, n)
        continued_fraction_suffix_convergent(coefficients, n) =
            Pair.new(continued_fraction_suffix_continuant(coefficients, n),
                continued_fraction_suffix_denominator(coefficients, n))
    }
}

/// A valid suffix has convergent method given by its continuant and
/// denominator methods.
theorem continued_fraction_suffix_convergent_eq_suffix_continuant_denominator_pair_of_valid_suffix(
    cf: ContinuedFraction, n: Nat
) {
    cf.has_valid_suffix(n)
        implies cf.suffix_convergent(n) =
            Pair.new(cf.suffix_continuant(n), cf.suffix_denominator(n))
} by {
    if cf.has_valid_suffix(n) {
        continued_fraction_has_valid_suffix_elim(cf, n)
        continued_fraction_suffix_convergent_eq_continuant_denominator_pair_of_valid_coefficients(
            cf.coefficients, n)
        continued_fraction_suffix_convergent_eq_coefficients_suffix_convergent(cf, n)
        continued_fraction_suffix_continuant_eq_coefficients_suffix_continuant(cf, n)
        continued_fraction_suffix_denominator_eq_coefficients_suffix_denominator(cf, n)
        cf.suffix_convergent(n) =
            Pair.new(cf.suffix_continuant(n), cf.suffix_denominator(n))
    }
}
