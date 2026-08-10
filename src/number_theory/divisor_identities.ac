from nat import Nat
from nat import pos_of_ne_zero, exp_ne_zero, lt_mul_both
from list import List, map, sum, product
from list import sum_map_of_pointwise, unique_same_contains_map_sum_eq
from pair import Pair, pair_first, pair_second
from number_theory.divisor_sum import divisor_list, divisor_sum_fn, divisor_sum_fn_apply,
    nat_sigma, nat_tau, nat_sigma_one, nat_tau_one, divisor_list_is_unique,
    sum_map_nat_identity_arithmetic_fn_eq_sum,
    divisor_sum_fn_nat_one_arithmetic_fn_eq_tau,
    divisor_sum_fn_nat_identity_arithmetic_fn_eq_sigma
from number_theory.arithmetic_functions import nat_identity_arithmetic_fn,
    nat_one_arithmetic_fn, nat_one_arithmetic_fn_apply
from number_theory.dirichlet import divisor_quotient, dirichlet_term, dirichlet_term_apply,
    dirichlet_convolve, dirichlet_convolve_apply, nat_divisor_quotient_fn,
    cofactor_image_list, cofactor_image_list_contains_iff, cofactor_image_list_is_unique
from number_theory.dirichlet_assoc import dirichlet_convolve_assoc
from number_theory.coprime import coprime_pow_pow
from number_theory.factorisation import all_prime, all_prime_cons_elim
from number_theory.pairwise_coprime import pairwise_coprime, pairwise_coprime_nil,
    pairwise_coprime_cons_imp, coprime_with_all,
    coprime_with_all_imp_coprime_product
from number_theory.tau_multiplicative import nat_tau_mul_coprime, nat_tau_prime_pow
from number_theory.sigma_multiplicative import nat_sigma_mul_coprime, nat_sigma_prime_pow
numerals Nat

// ---------------------------------------------------------------------------
// Dirichlet convolution against the constant-one function.
//
// The divisor sum `divisor_sum_fn(f)(n) = sum_{d | n} f(d)` is exactly the
// Dirichlet convolution of `f` with the constant-one arithmetic function, since
// the cofactor factor contributes `1(n / d) = 1` at every divisor.
// ---------------------------------------------------------------------------

/// Convolving `f` with the constant-one function is the divisor sum of `f`:
/// `(f * 1)(n) = sum_{d | n} f(d)`.
theorem dirichlet_convolve_one_right_eq_divisor_sum(f: Nat -> Nat, n: Nat) {
    dirichlet_convolve(f, nat_one_arithmetic_fn)(n) = divisor_sum_fn(f)(n)
} by {
    dirichlet_convolve_apply(f, nat_one_arithmetic_fn, n)
    dirichlet_convolve(f, nat_one_arithmetic_fn)(n) =
        sum(map(divisor_list(n), dirichlet_term(f, nat_one_arithmetic_fn, n)))
    divisor_sum_fn_apply(f, n)
    divisor_sum_fn(f)(n) = sum(map(divisor_list(n), f))
    forall(x: Nat) {
        if divisor_list(n).contains(x) {
            dirichlet_term_apply(f, nat_one_arithmetic_fn, n, x)
            dirichlet_term(f, nat_one_arithmetic_fn, n)(x) =
                f(x) * nat_one_arithmetic_fn(divisor_quotient(n, x))
            nat_one_arithmetic_fn_apply(divisor_quotient(n, x))
            nat_one_arithmetic_fn(divisor_quotient(n, x)) = Nat.1
            f(x) * Nat.1 = f(x)
            dirichlet_term(f, nat_one_arithmetic_fn, n)(x) = f(x)
        }
    }
    sum_map_of_pointwise(
        divisor_list(n), dirichlet_term(f, nat_one_arithmetic_fn, n), f)
    sum(map(divisor_list(n), dirichlet_term(f, nat_one_arithmetic_fn, n))) =
        sum(map(divisor_list(n), f))
    dirichlet_convolve(f, nat_one_arithmetic_fn)(n) =
        sum(map(divisor_list(n), f))
    dirichlet_convolve(f, nat_one_arithmetic_fn)(n) = divisor_sum_fn(f)(n)
}

/// `sigma = id * 1` pointwise: `sigma(n) = sum_{d | n} d`.
theorem nat_sigma_eq_dirichlet_convolve_id_one(n: Nat) {
    nat_sigma(n) = dirichlet_convolve(nat_identity_arithmetic_fn, nat_one_arithmetic_fn)(n)
} by {
    dirichlet_convolve_one_right_eq_divisor_sum(nat_identity_arithmetic_fn, n)
    dirichlet_convolve(nat_identity_arithmetic_fn, nat_one_arithmetic_fn)(n) =
        divisor_sum_fn(nat_identity_arithmetic_fn)(n)
    divisor_sum_fn_nat_identity_arithmetic_fn_eq_sigma(n)
    divisor_sum_fn(nat_identity_arithmetic_fn)(n) = nat_sigma(n)
    nat_sigma(n) = dirichlet_convolve(nat_identity_arithmetic_fn, nat_one_arithmetic_fn)(n)
}

/// `tau = 1 * 1` pointwise: `tau(n) = sum_{d | n} 1`.
theorem nat_tau_eq_dirichlet_convolve_one_one(n: Nat) {
    nat_tau(n) = dirichlet_convolve(nat_one_arithmetic_fn, nat_one_arithmetic_fn)(n)
} by {
    dirichlet_convolve_one_right_eq_divisor_sum(nat_one_arithmetic_fn, n)
    dirichlet_convolve(nat_one_arithmetic_fn, nat_one_arithmetic_fn)(n) =
        divisor_sum_fn(nat_one_arithmetic_fn)(n)
    divisor_sum_fn_nat_one_arithmetic_fn_eq_tau(n)
    divisor_sum_fn(nat_one_arithmetic_fn)(n) = nat_tau(n)
    nat_tau(n) = dirichlet_convolve(nat_one_arithmetic_fn, nat_one_arithmetic_fn)(n)
}

/// `sigma = id * 1` as an equality of arithmetic functions.
theorem nat_sigma_eq_dirichlet_convolve_id_one_fn {
    nat_sigma = dirichlet_convolve(nat_identity_arithmetic_fn, nat_one_arithmetic_fn)
} by {
    forall(n: Nat) {
        nat_sigma_eq_dirichlet_convolve_id_one(n)
        nat_sigma(n) = dirichlet_convolve(nat_identity_arithmetic_fn, nat_one_arithmetic_fn)(n)
    }
}

/// `tau = 1 * 1` as an equality of arithmetic functions.
theorem nat_tau_eq_dirichlet_convolve_one_one_fn {
    nat_tau = dirichlet_convolve(nat_one_arithmetic_fn, nat_one_arithmetic_fn)
} by {
    forall(n: Nat) {
        nat_tau_eq_dirichlet_convolve_one_one(n)
        nat_tau(n) = dirichlet_convolve(nat_one_arithmetic_fn, nat_one_arithmetic_fn)(n)
    }
}

// ---------------------------------------------------------------------------
// The classical convolution identity `sigma * 1 = id * tau`.
//
// Writing `sigma = id * 1` and `tau = 1 * 1`, associativity of Dirichlet
// convolution gives `sigma * 1 = (id * 1) * 1 = id * (1 * 1) = id * tau`,
// i.e. pointwise
//
//     sum_{d | n} sigma(d) = sum_{d | n} d * tau(n / d).
// ---------------------------------------------------------------------------

/// `sigma * 1 = id * tau` as an equality of arithmetic functions.
theorem nat_sigma_convolve_one_eq_id_convolve_tau_fn {
    dirichlet_convolve(nat_sigma, nat_one_arithmetic_fn) =
        dirichlet_convolve(nat_identity_arithmetic_fn, nat_tau)
} by {
    dirichlet_convolve_assoc(
        nat_identity_arithmetic_fn, nat_one_arithmetic_fn, nat_one_arithmetic_fn)
    dirichlet_convolve(nat_identity_arithmetic_fn,
        dirichlet_convolve(nat_one_arithmetic_fn, nat_one_arithmetic_fn)) =
        dirichlet_convolve(
            dirichlet_convolve(nat_identity_arithmetic_fn, nat_one_arithmetic_fn),
            nat_one_arithmetic_fn)
    nat_tau_eq_dirichlet_convolve_one_one_fn
    nat_tau = dirichlet_convolve(nat_one_arithmetic_fn, nat_one_arithmetic_fn)
    dirichlet_convolve(nat_identity_arithmetic_fn, nat_tau) =
        dirichlet_convolve(nat_identity_arithmetic_fn,
            dirichlet_convolve(nat_one_arithmetic_fn, nat_one_arithmetic_fn))
    nat_sigma_eq_dirichlet_convolve_id_one_fn
    nat_sigma = dirichlet_convolve(nat_identity_arithmetic_fn, nat_one_arithmetic_fn)
    dirichlet_convolve(nat_sigma, nat_one_arithmetic_fn) =
        dirichlet_convolve(
            dirichlet_convolve(nat_identity_arithmetic_fn, nat_one_arithmetic_fn),
            nat_one_arithmetic_fn)
    dirichlet_convolve(nat_sigma, nat_one_arithmetic_fn) =
        dirichlet_convolve(nat_identity_arithmetic_fn, nat_tau)
}

/// The classical convolution identity, in the divisor-sum form used by the
/// library: `sum_{d | n} sigma(d) = sum_{d | n} d * tau(n / d)`.  The right
/// hand side is the Dirichlet convolution of the identity with `tau`, since
/// the cofactor `n / d` is `divisor_quotient(n, d)`.
theorem nat_sigma_divisor_sum_eq_id_convolve_tau(n: Nat) {
    divisor_sum_fn(nat_sigma)(n) =
        dirichlet_convolve(nat_identity_arithmetic_fn, nat_tau)(n)
} by {
    dirichlet_convolve_one_right_eq_divisor_sum(nat_sigma, n)
    dirichlet_convolve(nat_sigma, nat_one_arithmetic_fn)(n) = divisor_sum_fn(nat_sigma)(n)
    nat_sigma_convolve_one_eq_id_convolve_tau_fn
    dirichlet_convolve(nat_sigma, nat_one_arithmetic_fn) =
        dirichlet_convolve(nat_identity_arithmetic_fn, nat_tau)
    dirichlet_convolve(nat_sigma, nat_one_arithmetic_fn)(n) =
        dirichlet_convolve(nat_identity_arithmetic_fn, nat_tau)(n)
    divisor_sum_fn(nat_sigma)(n) =
        dirichlet_convolve(nat_identity_arithmetic_fn, nat_tau)(n)
}

/// The same identity with the right hand side unfolded to the divisor-list
/// sum: `sum_{d | n} sigma(d) = sum_{d | n} d * tau(n / d)`.
theorem nat_sigma_divisor_sum_eq_id_convolve_tau_sum(n: Nat) {
    divisor_sum_fn(nat_sigma)(n) =
        sum(map(divisor_list(n),
            dirichlet_term(nat_identity_arithmetic_fn, nat_tau, n)))
} by {
    nat_sigma_divisor_sum_eq_id_convolve_tau(n)
    dirichlet_convolve_apply(nat_identity_arithmetic_fn, nat_tau, n)
    dirichlet_convolve(nat_identity_arithmetic_fn, nat_tau)(n) =
        sum(map(divisor_list(n),
            dirichlet_term(nat_identity_arithmetic_fn, nat_tau, n)))
    divisor_sum_fn(nat_sigma)(n) =
        sum(map(divisor_list(n),
            dirichlet_term(nat_identity_arithmetic_fn, nat_tau, n)))
}

// ---------------------------------------------------------------------------
// The divisor pairing `sum_{d | n} d = sum_{d | n} n / d`.
//
// The cofactor map `d -> n / d` permutes the divisors of positive `n` (it is
// an involution by `divisor_quotient_involution`), so summing the identity
// over the cofactor image gives the same value as summing it over the divisor
// list, namely `sigma(n)`.
// ---------------------------------------------------------------------------

/// `sum_{d | n} n / d = sigma(n)` for positive `n`: the cofactor image of the
/// divisor list has the same members as the divisor list itself.
theorem nat_sigma_eq_cofactor_image_sum(n: Nat) {
    Nat.0 < n implies nat_sigma(n) = sum(cofactor_image_list(n))
} by {
    if Nat.0 < n {
        cofactor_image_list_is_unique(n)
        cofactor_image_list(n).is_unique
        divisor_list_is_unique(n)
        divisor_list(n).is_unique
        forall(x: Nat) {
            cofactor_image_list_contains_iff(n, x)
            cofactor_image_list(n).contains(x) = divisor_list(n).contains(x)
        }
        unique_same_contains_map_sum_eq(
            cofactor_image_list(n), divisor_list(n), nat_identity_arithmetic_fn)
        sum(map(cofactor_image_list(n), nat_identity_arithmetic_fn)) =
            sum(map(divisor_list(n), nat_identity_arithmetic_fn))
        sum_map_nat_identity_arithmetic_fn_eq_sum(cofactor_image_list(n))
        sum(map(cofactor_image_list(n), nat_identity_arithmetic_fn)) =
            sum(cofactor_image_list(n))
        sum_map_nat_identity_arithmetic_fn_eq_sum(divisor_list(n))
        sum(map(divisor_list(n), nat_identity_arithmetic_fn)) = sum(divisor_list(n))
        sum(cofactor_image_list(n)) = sum(divisor_list(n))
        nat_sigma(n) = sum(divisor_list(n))
        nat_sigma(n) = sum(cofactor_image_list(n))
    }
}

/// The divisor pairing in the divisor-sum notation of the library:
/// `sum_{d | n} d = sum_{d | n} n / d` for positive `n`, where the cofactor
/// `n / d` is `divisor_quotient(n, d)`.
theorem nat_sigma_eq_divisor_sum_cofactor(n: Nat) {
    Nat.0 < n implies
        nat_sigma(n) = divisor_sum_fn(nat_divisor_quotient_fn(n))(n)
} by {
    if Nat.0 < n {
        nat_sigma_eq_cofactor_image_sum(n)
        nat_sigma(n) = sum(cofactor_image_list(n))
        divisor_sum_fn_apply(nat_divisor_quotient_fn(n), n)
        divisor_sum_fn(nat_divisor_quotient_fn(n))(n) =
            sum(map(divisor_list(n), nat_divisor_quotient_fn(n)))
        cofactor_image_list(n) = map(divisor_list(n), nat_divisor_quotient_fn(n))
        sum(map(divisor_list(n), nat_divisor_quotient_fn(n))) =
            sum(cofactor_image_list(n))
        nat_sigma(n) = divisor_sum_fn(nat_divisor_quotient_fn(n))(n)
    }
}

// ---------------------------------------------------------------------------
// The product formulas for tau and sigma.
//
// A list of pairwise coprime arguments supports multiplicative product
// formulas: tau(product(list)) = product(map(list, tau)) and the same for
// sigma.  Applied to the prime-power decomposition n = prod p_i^{e_i} (with
// the p_i distinct primes) and the prime-power values tau(p^e) = e + 1 and
// sigma(p^e) = (p^(e+1) - 1) / (p - 1), these give the classical product
// formulas
//
//     tau(n) = prod_i (e_i + 1)          sigma(n) = prod_i (p_i^(e_i+1) - 1)/(p_i - 1).
//
// The statements below use a list of pairs (p_i, e_i) so that both the prime
// bases and the exponents are carried along.
// ---------------------------------------------------------------------------

/// True if every element of the list is a positive natural.
define all_positive(list: List[Nat]) -> Bool {
    match list {
        List.nil {
            true
        }
        List.cons(head, tail) {
            Nat.0 < head and all_positive(tail)
        }
    }
}

/// The power of a pair `(p, e)`, namely `p^e`.
define pair_pow(p: Pair[Nat, Nat]) -> Nat {
    p.first.pow(p.second)
}

/// The successor of the exponent of a pair `(p, e)`, namely `e + 1`.
define pair_second_suc(p: Pair[Nat, Nat]) -> Nat {
    p.second + Nat.1
}

/// The sigma value of a prime power: `(p^(e+1) - 1) / (p - 1)`.
define pair_sigma_factor(p: Pair[Nat, Nat]) -> Nat {
    (p.first.pow(p.second + Nat.1) - Nat.1).div(p.first - Nat.1)
}

/// The product of a list of positive naturals is positive.
theorem product_all_positive(list: List[Nat]) {
    all_positive(list) implies Nat.0 < product[Nat](list)
} by {
    define p(xs: List[Nat]) -> Bool {
        all_positive(xs) implies Nat.0 < product[Nat](xs)
    }
    product[Nat](List.nil[Nat]) = Nat.1
    Nat.0 < Nat.1
    p(List.nil[Nat])
    forall(head: Nat, tail: List[Nat]) {
        if p(tail) {
            if all_positive(List.cons(head, tail)) {
                Nat.0 < head
                all_positive(tail)
                p(tail)
                Nat.0 < product[Nat](tail)
                head != Nat.0
                lt_mul_both(head, Nat.0, product[Nat](tail))
                head * Nat.0 < head * product[Nat](tail)
                head * Nat.0 = Nat.0
                Nat.0 < head * product[Nat](tail)
                product[Nat](List.cons(head, tail)) = head * product[Nat](tail)
                Nat.0 < product[Nat](List.cons(head, tail))
            }
            p(List.cons(head, tail))
        }
    }
    forall(head: Nat, tail: List[Nat]) {
        p(tail) implies p(List.cons(head, tail))
    }
    p(List.nil[Nat]) and forall(head: Nat, tail: List[Nat]) {
        p(tail) implies p(List.cons(head, tail))
    }
    List.induction(p)
    forall(ls: List[Nat]) { p(ls) }
    p(list)
}

/// `tau` is multiplicative over a pairwise coprime list:
/// `tau(product(list)) = product(map(list, tau))`.
theorem nat_tau_mul_pairwise_coprime(ps: List[Nat]) {
    pairwise_coprime(ps) implies nat_tau(product[Nat](ps)) = product(map(ps, nat_tau))
} by {
    define p(xs: List[Nat]) -> Bool {
        pairwise_coprime(xs) implies nat_tau(product[Nat](xs)) = product(map(xs, nat_tau))
    }
    pairwise_coprime_nil
    pairwise_coprime(List.nil[Nat])
    nat_tau_one
    nat_tau(Nat.1) = Nat.1
    product[Nat](List.nil[Nat]) = Nat.1
    nat_tau(product[Nat](List.nil[Nat])) = Nat.1
    product(map(List.nil[Nat], nat_tau)) = Nat.1
    nat_tau(product[Nat](List.nil[Nat])) = product(map(List.nil[Nat], nat_tau))
    p(List.nil[Nat])
    forall(head: Nat, tail: List[Nat]) {
        if p(tail) {
            if pairwise_coprime(List.cons(head, tail)) {
                pairwise_coprime_cons_imp(head, tail)
                coprime_with_all(head, tail) and pairwise_coprime(tail)
                coprime_with_all_imp_coprime_product(head, tail)
                head.coprime(product[Nat](tail))
                nat_tau_mul_coprime(head, product[Nat](tail))
                nat_tau(head * product[Nat](tail)) =
                    nat_tau(head) * nat_tau(product[Nat](tail))
                product[Nat](List.cons(head, tail)) = head * product[Nat](tail)
                nat_tau(product[Nat](List.cons(head, tail))) =
                    nat_tau(head * product[Nat](tail))
                p(tail)
                nat_tau(product[Nat](tail)) = product(map(tail, nat_tau))
                nat_tau(head * product[Nat](tail)) =
                    nat_tau(head) * product(map(tail, nat_tau))
                map(List.cons(head, tail), nat_tau) =
                    List.cons(nat_tau(head), map(tail, nat_tau))
                product(List.cons(nat_tau(head), map(tail, nat_tau))) =
                    nat_tau(head) * product(map(tail, nat_tau))
                product(map(List.cons(head, tail), nat_tau)) =
                    nat_tau(head) * product(map(tail, nat_tau))
                nat_tau(product[Nat](List.cons(head, tail))) =
                    product(map(List.cons(head, tail), nat_tau))
            }
            p(List.cons(head, tail))
        }
    }
    forall(head: Nat, tail: List[Nat]) {
        p(tail) implies p(List.cons(head, tail))
    }
    p(List.nil[Nat]) and forall(head: Nat, tail: List[Nat]) {
        p(tail) implies p(List.cons(head, tail))
    }
    List.induction(p)
    forall(ls: List[Nat]) { p(ls) }
    p(ps)
}

/// `sigma` is multiplicative over a pairwise coprime list of positive naturals:
/// `sigma(product(list)) = product(map(list, sigma))`.
theorem nat_sigma_mul_pairwise_coprime(ps: List[Nat]) {
    pairwise_coprime(ps) and all_positive(ps)
        implies nat_sigma(product[Nat](ps)) = product(map(ps, nat_sigma))
} by {
    define p(xs: List[Nat]) -> Bool {
        pairwise_coprime(xs) and all_positive(xs)
            implies nat_sigma(product[Nat](xs)) = product(map(xs, nat_sigma))
    }
    nat_sigma_one
    nat_sigma(Nat.1) = Nat.1
    product[Nat](List.nil[Nat]) = Nat.1
    nat_sigma(product[Nat](List.nil[Nat])) = Nat.1
    product(map(List.nil[Nat], nat_sigma)) = Nat.1
    nat_sigma(product[Nat](List.nil[Nat])) = product(map(List.nil[Nat], nat_sigma))
    p(List.nil[Nat])
    forall(head: Nat, tail: List[Nat]) {
        if p(tail) {
            if pairwise_coprime(List.cons(head, tail)) and all_positive(List.cons(head, tail)) {
                pairwise_coprime_cons_imp(head, tail)
                coprime_with_all(head, tail) and pairwise_coprime(tail)
                Nat.0 < head
                all_positive(tail)
                coprime_with_all_imp_coprime_product(head, tail)
                head.coprime(product[Nat](tail))
                product_all_positive(tail)
                Nat.0 < product[Nat](tail)
                nat_sigma_mul_coprime(head, product[Nat](tail))
                nat_sigma(head * product[Nat](tail)) =
                    nat_sigma(head) * nat_sigma(product[Nat](tail))
                product[Nat](List.cons(head, tail)) = head * product[Nat](tail)
                nat_sigma(product[Nat](List.cons(head, tail))) =
                    nat_sigma(head * product[Nat](tail))
                p(tail)
                nat_sigma(product[Nat](tail)) = product(map(tail, nat_sigma))
                nat_sigma(head * product[Nat](tail)) =
                    nat_sigma(head) * product(map(tail, nat_sigma))
                map(List.cons(head, tail), nat_sigma) =
                    List.cons(nat_sigma(head), map(tail, nat_sigma))
                product(List.cons(nat_sigma(head), map(tail, nat_sigma))) =
                    nat_sigma(head) * product(map(tail, nat_sigma))
                product(map(List.cons(head, tail), nat_sigma)) =
                    nat_sigma(head) * product(map(tail, nat_sigma))
                nat_sigma(product[Nat](List.cons(head, tail))) =
                    product(map(List.cons(head, tail), nat_sigma))
            }
            p(List.cons(head, tail))
        }
    }
    forall(head: Nat, tail: List[Nat]) {
        p(tail) implies p(List.cons(head, tail))
    }
    p(List.nil[Nat]) and forall(head: Nat, tail: List[Nat]) {
        p(tail) implies p(List.cons(head, tail))
    }
    List.induction(p)
    forall(ls: List[Nat]) { p(ls) }
    p(ps)
}

/// Powering the first coordinates preserves coprimality with every element of
/// a list of pairs: if `a` is coprime with each base, then `a^e` is coprime
/// with each corresponding prime power.
theorem coprime_with_all_pair_pow(a: Nat, e: Nat, tail: List[Pair[Nat, Nat]]) {
    coprime_with_all(a, map(tail, pair_first[Nat, Nat]))
        implies coprime_with_all(a.pow(e), map(tail, pair_pow))
} by {
    define p(xs: List[Pair[Nat, Nat]]) -> Bool {
        coprime_with_all(a, map(xs, pair_first[Nat, Nat]))
            implies coprime_with_all(a.pow(e), map(xs, pair_pow))
    }
    map(List.nil[Pair[Nat, Nat]], pair_first[Nat, Nat]) = List.nil[Nat]
    map(List.nil[Pair[Nat, Nat]], pair_pow) = List.nil[Nat]
    if coprime_with_all(a, map(List.nil[Pair[Nat, Nat]], pair_first[Nat, Nat])) {
        coprime_with_all(a.pow(e), List.nil[Nat])
        coprime_with_all(a.pow(e), map(List.nil[Pair[Nat, Nat]], pair_pow))
    }
    p(List.nil[Pair[Nat, Nat]])
    forall(head: Pair[Nat, Nat], tail2: List[Pair[Nat, Nat]]) {
        if p(tail2) {
            if coprime_with_all(a, map(List.cons(head, tail2), pair_first[Nat, Nat])) {
                map(List.cons(head, tail2), pair_first[Nat, Nat]) =
                    List.cons(pair_first[Nat, Nat](head), map(tail2, pair_first[Nat, Nat]))
                coprime_with_all(a,
                    List.cons(pair_first[Nat, Nat](head), map(tail2, pair_first[Nat, Nat])))
                a.coprime(pair_first[Nat, Nat](head))
                a.coprime(head.first)
                coprime_with_all(a, map(tail2, pair_first[Nat, Nat]))
                coprime_pow_pow(a, head.first, e, head.second)
                a.pow(e).coprime(head.first.pow(head.second))
                p(tail2)
                coprime_with_all(a.pow(e), map(tail2, pair_pow))
                pair_pow(head) = head.first.pow(head.second)
                a.pow(e).coprime(pair_pow(head))
                coprime_with_all(a.pow(e),
                    List.cons(pair_pow(head), map(tail2, pair_pow)))
                map(List.cons(head, tail2), pair_pow) =
                    List.cons(pair_pow(head), map(tail2, pair_pow))
                coprime_with_all(a.pow(e), map(List.cons(head, tail2), pair_pow))
            }
            p(List.cons(head, tail2))
        }
    }
    forall(head: Pair[Nat, Nat], tail2: List[Pair[Nat, Nat]]) {
        p(tail2) implies p(List.cons(head, tail2))
    }
    p(List.nil[Pair[Nat, Nat]]) and forall(head: Pair[Nat, Nat], tail2: List[Pair[Nat, Nat]]) {
        p(tail2) implies p(List.cons(head, tail2))
    }
    List.induction(p)
    forall(ls: List[Pair[Nat, Nat]]) { p(ls) }
    p(tail)
}

/// The prime powers of pairwise coprime bases are pairwise coprime.
theorem prime_power_list_pairwise_coprime(pairs: List[Pair[Nat, Nat]]) {
    pairwise_coprime(map(pairs, pair_first[Nat, Nat]))
        implies pairwise_coprime(map(pairs, pair_pow))
} by {
    define p(xs: List[Pair[Nat, Nat]]) -> Bool {
        pairwise_coprime(map(xs, pair_first[Nat, Nat]))
            implies pairwise_coprime(map(xs, pair_pow))
    }
    map(List.nil[Pair[Nat, Nat]], pair_first[Nat, Nat]) = List.nil[Nat]
    map(List.nil[Pair[Nat, Nat]], pair_pow) = List.nil[Nat]
    if pairwise_coprime(map(List.nil[Pair[Nat, Nat]], pair_first[Nat, Nat])) {
        pairwise_coprime(List.nil[Nat])
        pairwise_coprime(map(List.nil[Pair[Nat, Nat]], pair_pow))
    }
    p(List.nil[Pair[Nat, Nat]])
    forall(head: Pair[Nat, Nat], tail2: List[Pair[Nat, Nat]]) {
        if p(tail2) {
            if pairwise_coprime(map(List.cons(head, tail2), pair_first[Nat, Nat])) {
                map(List.cons(head, tail2), pair_first[Nat, Nat]) =
                    List.cons(pair_first[Nat, Nat](head), map(tail2, pair_first[Nat, Nat]))
                pairwise_coprime(
                    List.cons(pair_first[Nat, Nat](head), map(tail2, pair_first[Nat, Nat])))
                pairwise_coprime_cons_imp(pair_first[Nat, Nat](head),
                    map(tail2, pair_first[Nat, Nat]))
                coprime_with_all(pair_first[Nat, Nat](head), map(tail2, pair_first[Nat, Nat])) and
                    pairwise_coprime(map(tail2, pair_first[Nat, Nat]))
                coprime_with_all_pair_pow(pair_first[Nat, Nat](head),
                    pair_second[Nat, Nat](head), tail2)
                coprime_with_all(pair_first[Nat, Nat](head).pow(pair_second[Nat, Nat](head)),
                    map(tail2, pair_pow))
                p(tail2)
                pairwise_coprime(map(tail2, pair_pow))
                pair_pow(head) = pair_first[Nat, Nat](head).pow(pair_second[Nat, Nat](head))
                coprime_with_all(pair_pow(head), map(tail2, pair_pow))
                pairwise_coprime(List.cons(pair_pow(head), map(tail2, pair_pow)))
                map(List.cons(head, tail2), pair_pow) =
                    List.cons(pair_pow(head), map(tail2, pair_pow))
                pairwise_coprime(map(List.cons(head, tail2), pair_pow))
            }
            p(List.cons(head, tail2))
        }
    }
    forall(head: Pair[Nat, Nat], tail2: List[Pair[Nat, Nat]]) {
        p(tail2) implies p(List.cons(head, tail2))
    }
    p(List.nil[Pair[Nat, Nat]]) and forall(head: Pair[Nat, Nat], tail2: List[Pair[Nat, Nat]]) {
        p(tail2) implies p(List.cons(head, tail2))
    }
    List.induction(p)
    forall(ls: List[Pair[Nat, Nat]]) { p(ls) }
    p(pairs)
}

/// Prime powers of primes are positive.
theorem prime_power_list_all_positive(pairs: List[Pair[Nat, Nat]]) {
    all_prime(map(pairs, pair_first[Nat, Nat])) implies all_positive(map(pairs, pair_pow))
} by {
    define p(xs: List[Pair[Nat, Nat]]) -> Bool {
        all_prime(map(xs, pair_first[Nat, Nat])) implies all_positive(map(xs, pair_pow))
    }
    map(List.nil[Pair[Nat, Nat]], pair_first[Nat, Nat]) = List.nil[Nat]
    map(List.nil[Pair[Nat, Nat]], pair_pow) = List.nil[Nat]
    if all_prime(map(List.nil[Pair[Nat, Nat]], pair_first[Nat, Nat])) {
        all_positive(List.nil[Nat])
        all_positive(map(List.nil[Pair[Nat, Nat]], pair_pow))
    }
    p(List.nil[Pair[Nat, Nat]])
    forall(head: Pair[Nat, Nat], tail2: List[Pair[Nat, Nat]]) {
        if p(tail2) {
            if all_prime(map(List.cons(head, tail2), pair_first[Nat, Nat])) {
                map(List.cons(head, tail2), pair_first[Nat, Nat]) =
                    List.cons(pair_first[Nat, Nat](head), map(tail2, pair_first[Nat, Nat]))
                all_prime(List.cons(pair_first[Nat, Nat](head), map(tail2, pair_first[Nat, Nat])))
                all_prime_cons_elim(pair_first[Nat, Nat](head), map(tail2, pair_first[Nat, Nat]))
                pair_first[Nat, Nat](head).is_prime and all_prime(map(tail2, pair_first[Nat, Nat]))
                head.first.is_prime
                Nat.1 < head.first
                head.first != Nat.0
                exp_ne_zero(head.first, head.second)
                head.first.pow(head.second) != Nat.0
                pos_of_ne_zero(head.first.pow(head.second))
                Nat.0 < head.first.pow(head.second)
                p(tail2)
                all_positive(map(tail2, pair_pow))
                all_positive(List.cons(head.first.pow(head.second), map(tail2, pair_pow)))
                pair_pow(head) = head.first.pow(head.second)
                map(List.cons(head, tail2), pair_pow) =
                    List.cons(pair_pow(head), map(tail2, pair_pow))
                all_positive(map(List.cons(head, tail2), pair_pow))
            }
            p(List.cons(head, tail2))
        }
    }
    forall(head: Pair[Nat, Nat], tail2: List[Pair[Nat, Nat]]) {
        p(tail2) implies p(List.cons(head, tail2))
    }
    p(List.nil[Pair[Nat, Nat]]) and forall(head: Pair[Nat, Nat], tail2: List[Pair[Nat, Nat]]) {
        p(tail2) implies p(List.cons(head, tail2))
    }
    List.induction(p)
    forall(ls: List[Pair[Nat, Nat]]) { p(ls) }
    p(pairs)
}

/// Mapping `tau` over the prime powers `p^e` of a list of prime bases gives
/// the list of `e + 1`.
theorem map_pow_nat_tau_eq_second_suc(pairs: List[Pair[Nat, Nat]]) {
    all_prime(map(pairs, pair_first[Nat, Nat])) implies
        map(map(pairs, pair_pow), nat_tau) = map(pairs, pair_second_suc)
} by {
    define p(xs: List[Pair[Nat, Nat]]) -> Bool {
        all_prime(map(xs, pair_first[Nat, Nat])) implies
            map(map(xs, pair_pow), nat_tau) = map(xs, pair_second_suc)
    }
    map(List.nil[Pair[Nat, Nat]], pair_second_suc) = List.nil[Nat]
    if all_prime(map(List.nil[Pair[Nat, Nat]], pair_first[Nat, Nat])) {
        map(List.nil[Nat], nat_tau) = List.nil[Nat]
        map(map(List.nil[Pair[Nat, Nat]], pair_pow), nat_tau) = List.nil[Nat]
        map(map(List.nil[Pair[Nat, Nat]], pair_pow), nat_tau) =
            map(List.nil[Pair[Nat, Nat]], pair_second_suc)
    }
    p(List.nil[Pair[Nat, Nat]])
    forall(head: Pair[Nat, Nat], tail2: List[Pair[Nat, Nat]]) {
        if p(tail2) {
            if all_prime(map(List.cons(head, tail2), pair_first[Nat, Nat])) {
                map(List.cons(head, tail2), pair_first[Nat, Nat]) =
                    List.cons(pair_first[Nat, Nat](head), map(tail2, pair_first[Nat, Nat]))
                all_prime(List.cons(pair_first[Nat, Nat](head), map(tail2, pair_first[Nat, Nat])))
                all_prime_cons_elim(pair_first[Nat, Nat](head), map(tail2, pair_first[Nat, Nat]))
                head.first.is_prime
                nat_tau_prime_pow(head.first, head.second)
                nat_tau(head.first.pow(head.second)) = head.second + Nat.1
                map(List.cons(head, tail2), pair_pow) =
                    List.cons(pair_pow(head), map(tail2, pair_pow))
                map(map(List.cons(head, tail2), pair_pow), nat_tau) =
                    map(List.cons(pair_pow(head), map(tail2, pair_pow)), nat_tau)
                map(List.cons(pair_pow(head), map(tail2, pair_pow)), nat_tau) =
                    List.cons(nat_tau(pair_pow(head)),
                        map(map(tail2, pair_pow), nat_tau))
                pair_pow(head) = head.first.pow(head.second)
                nat_tau(pair_pow(head)) = nat_tau(head.first.pow(head.second))
                nat_tau(pair_pow(head)) = head.second + Nat.1
                p(tail2)
                map(map(tail2, pair_pow), nat_tau) = map(tail2, pair_second_suc)
                List.cons(nat_tau(pair_pow(head)), map(map(tail2, pair_pow), nat_tau)) =
                    List.cons(head.second + Nat.1, map(tail2, pair_second_suc))
                map(List.cons(head, tail2), pair_second_suc) =
                    List.cons(pair_second_suc(head), map(tail2, pair_second_suc))
                pair_second_suc(head) = head.second + Nat.1
                map(map(List.cons(head, tail2), pair_pow), nat_tau) =
                    map(List.cons(head, tail2), pair_second_suc)
            }
            p(List.cons(head, tail2))
        }
    }
    forall(head: Pair[Nat, Nat], tail2: List[Pair[Nat, Nat]]) {
        p(tail2) implies p(List.cons(head, tail2))
    }
    p(List.nil[Pair[Nat, Nat]]) and forall(head: Pair[Nat, Nat], tail2: List[Pair[Nat, Nat]]) {
        p(tail2) implies p(List.cons(head, tail2))
    }
    List.induction(p)
    forall(ls: List[Pair[Nat, Nat]]) { p(ls) }
    p(pairs)
}

/// Mapping `sigma` over the prime powers `p^e` of a list of prime bases gives
/// the list of `(p^(e+1) - 1) / (p - 1)`.
theorem map_pow_nat_sigma_eq_sigma_factor(pairs: List[Pair[Nat, Nat]]) {
    all_prime(map(pairs, pair_first[Nat, Nat])) implies
        map(map(pairs, pair_pow), nat_sigma) = map(pairs, pair_sigma_factor)
} by {
    define p(xs: List[Pair[Nat, Nat]]) -> Bool {
        all_prime(map(xs, pair_first[Nat, Nat])) implies
            map(map(xs, pair_pow), nat_sigma) = map(xs, pair_sigma_factor)
    }
    map(List.nil[Pair[Nat, Nat]], pair_sigma_factor) = List.nil[Nat]
    if all_prime(map(List.nil[Pair[Nat, Nat]], pair_first[Nat, Nat])) {
        map(List.nil[Nat], nat_sigma) = List.nil[Nat]
        map(map(List.nil[Pair[Nat, Nat]], pair_pow), nat_sigma) = List.nil[Nat]
        map(map(List.nil[Pair[Nat, Nat]], pair_pow), nat_sigma) =
            map(List.nil[Pair[Nat, Nat]], pair_sigma_factor)
    }
    p(List.nil[Pair[Nat, Nat]])
    forall(head: Pair[Nat, Nat], tail2: List[Pair[Nat, Nat]]) {
        if p(tail2) {
            if all_prime(map(List.cons(head, tail2), pair_first[Nat, Nat])) {
                map(List.cons(head, tail2), pair_first[Nat, Nat]) =
                    List.cons(pair_first[Nat, Nat](head), map(tail2, pair_first[Nat, Nat]))
                all_prime(List.cons(pair_first[Nat, Nat](head), map(tail2, pair_first[Nat, Nat])))
                all_prime_cons_elim(pair_first[Nat, Nat](head), map(tail2, pair_first[Nat, Nat]))
                head.first.is_prime
                nat_sigma_prime_pow(head.first, head.second)
                nat_sigma(head.first.pow(head.second)) =
                    (head.first.pow(head.second + Nat.1) - Nat.1).div(head.first - Nat.1)
                map(List.cons(head, tail2), pair_pow) =
                    List.cons(pair_pow(head), map(tail2, pair_pow))
                map(map(List.cons(head, tail2), pair_pow), nat_sigma) =
                    map(List.cons(pair_pow(head), map(tail2, pair_pow)), nat_sigma)
                map(List.cons(pair_pow(head), map(tail2, pair_pow)), nat_sigma) =
                    List.cons(nat_sigma(pair_pow(head)),
                        map(map(tail2, pair_pow), nat_sigma))
                pair_pow(head) = head.first.pow(head.second)
                nat_sigma(pair_pow(head)) = nat_sigma(head.first.pow(head.second))
                pair_sigma_factor(head) =
                    (head.first.pow(head.second + Nat.1) - Nat.1).div(head.first - Nat.1)
                nat_sigma(pair_pow(head)) = pair_sigma_factor(head)
                p(tail2)
                map(map(tail2, pair_pow), nat_sigma) = map(tail2, pair_sigma_factor)
                List.cons(nat_sigma(pair_pow(head)),
                    map(map(tail2, pair_pow), nat_sigma)) =
                    List.cons(pair_sigma_factor(head), map(tail2, pair_sigma_factor))
                map(List.cons(head, tail2), pair_sigma_factor) =
                    List.cons(pair_sigma_factor(head), map(tail2, pair_sigma_factor))
                map(map(List.cons(head, tail2), pair_pow), nat_sigma) =
                    map(List.cons(head, tail2), pair_sigma_factor)
            }
            p(List.cons(head, tail2))
        }
    }
    forall(head: Pair[Nat, Nat], tail2: List[Pair[Nat, Nat]]) {
        p(tail2) implies p(List.cons(head, tail2))
    }
    p(List.nil[Pair[Nat, Nat]]) and forall(head: Pair[Nat, Nat], tail2: List[Pair[Nat, Nat]]) {
        p(tail2) implies p(List.cons(head, tail2))
    }
    List.induction(p)
    forall(ls: List[Pair[Nat, Nat]]) { p(ls) }
    p(pairs)
}

/// The product formula for `tau`: if `n = prod p_i^(e_i)` with the `p_i`
/// distinct primes, then `tau(n) = prod (e_i + 1)`.
theorem nat_tau_prime_power_product_formula(pairs: List[Pair[Nat, Nat]], n: Nat) {
    all_prime(map(pairs, pair_first[Nat, Nat])) and
        pairwise_coprime(map(pairs, pair_first[Nat, Nat])) and
        product[Nat](map(pairs, pair_pow)) = n
        implies nat_tau(n) = product(map(pairs, pair_second_suc))
} by {
    if all_prime(map(pairs, pair_first[Nat, Nat])) and
            pairwise_coprime(map(pairs, pair_first[Nat, Nat])) and
            product[Nat](map(pairs, pair_pow)) = n {
        prime_power_list_pairwise_coprime(pairs)
        pairwise_coprime(map(pairs, pair_pow))
        nat_tau_mul_pairwise_coprime(map(pairs, pair_pow))
        nat_tau(product[Nat](map(pairs, pair_pow))) =
            product(map(map(pairs, pair_pow), nat_tau))
        product[Nat](map(pairs, pair_pow)) = n
        nat_tau(n) = product(map(map(pairs, pair_pow), nat_tau))
        map_pow_nat_tau_eq_second_suc(pairs)
        map(map(pairs, pair_pow), nat_tau) = map(pairs, pair_second_suc)
        product(map(map(pairs, pair_pow), nat_tau)) =
            product(map(pairs, pair_second_suc))
        nat_tau(n) = product(map(pairs, pair_second_suc))
    }
}

/// The product formula for `sigma`: if `n = prod p_i^(e_i)` with the `p_i`
/// distinct primes, then `sigma(n) = prod (p_i^(e_i+1) - 1) / (p_i - 1)`.
theorem nat_sigma_prime_power_product_formula(pairs: List[Pair[Nat, Nat]], n: Nat) {
    all_prime(map(pairs, pair_first[Nat, Nat])) and
        pairwise_coprime(map(pairs, pair_first[Nat, Nat])) and
        product[Nat](map(pairs, pair_pow)) = n
        implies nat_sigma(n) = product(map(pairs, pair_sigma_factor))
} by {
    if all_prime(map(pairs, pair_first[Nat, Nat])) and
            pairwise_coprime(map(pairs, pair_first[Nat, Nat])) and
            product[Nat](map(pairs, pair_pow)) = n {
        prime_power_list_pairwise_coprime(pairs)
        pairwise_coprime(map(pairs, pair_pow))
        prime_power_list_all_positive(pairs)
        all_positive(map(pairs, pair_pow))
        nat_sigma_mul_pairwise_coprime(map(pairs, pair_pow))
        nat_sigma(product[Nat](map(pairs, pair_pow))) =
            product(map(map(pairs, pair_pow), nat_sigma))
        product[Nat](map(pairs, pair_pow)) = n
        nat_sigma(n) = product(map(map(pairs, pair_pow), nat_sigma))
        map_pow_nat_sigma_eq_sigma_factor(pairs)
        map(map(pairs, pair_pow), nat_sigma) = map(pairs, pair_sigma_factor)
        product(map(map(pairs, pair_pow), nat_sigma)) =
            product(map(pairs, pair_sigma_factor))
        nat_sigma(n) = product(map(pairs, pair_sigma_factor))
    }
}

// ---------------------------------------------------------------------------
// Oddness of sigma.
//
// The classical characterisation: `sigma(n)` is odd if and only if `n` is a
// square or twice a square.  Proving it needs the parity theory of the
// naturals (mod-2 arithmetic) together with the product formula above:
// `sigma(n) = prod sigma(p^e)`, and `sigma(p^e) = 1 + p + ... + p^e` is odd
// exactly when `p = 2` or (`p` odd and `e` even), so `sigma(n)` is odd exactly
// when every odd prime divides `n` to an even power, i.e. `n` is a square or
// twice a square.  The library has no even/odd development for the naturals
// yet, so the statement is left here as a comment rather than a theorem.
//
// define nat_is_odd(n: Nat) -> Bool {
//     exists(k: Nat) { n = Nat.2 * k + Nat.1 }
// }
//
// theorem nat_sigma_odd_iff_square_or_twice_square(n: Nat) {
//     nat_is_odd(nat_sigma(n)) = (is_square(n) or exists(k: Nat) { n = Nat.2 * (k * k) })
// }
