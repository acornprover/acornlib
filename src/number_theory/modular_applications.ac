/// Modular arithmetic applications.
///
/// This file collects the classical applications of the congruence machinery
/// as standalone statements: Euler's theorem, Wilson's theorem, the
/// divisibility of the multiplicative order by the totient, and Fermat's
/// little theorem in its arbitrary-base and base-two forms.
///
/// Each theorem below is proved in its home file — `totient.ac` (Euler),
/// `wilson.ac` (Wilson), `multiplicative_order.ac` (order divides totient),
/// and `fermat.ac` (Fermat) — and is restated here in one place with a
/// citation to the original proof.

from nat import Nat
from number_theory.congruence import congr_mod_refl, congr_mod_symm,
    congr_mod_trans, congr_mod_mul
from number_theory.totient import euler, coprime_below_prime
from number_theory.wilson import prime_imp_wilson_factorial_congr
from number_theory.multiplicative_order import multiplicative_order_mod,
    multiplicative_order_mod_divides_totient
from number_theory.fermat import fermats_little_congr, fermat_euler
numerals Nat

// ---------------------------------------------------------------------------
// Euler's theorem.
// ---------------------------------------------------------------------------

/// Euler's theorem: for `n > 0` and `a` coprime to `n`, the power
/// `a^φ(n)` is congruent to `1` modulo `n`, where `φ` is Euler's totient.
/// This is the statement of `euler` from totient.ac, restated here as a
/// standalone theorem.
theorem euler_theorem_general(n: Nat, a: Nat) {
    n != Nat.0 and a.coprime(n) implies a.pow(n.totient).congr_mod(Nat.1, n)
} by {
    if n != Nat.0 and a.coprime(n) {
        euler(n, a)
        a.pow(n.totient).congr_mod(Nat.1, n)
    }
}

// ---------------------------------------------------------------------------
// Wilson's theorem.
// ---------------------------------------------------------------------------

/// Wilson's theorem: for a prime `p`, the factorial `(p - 1)!` is congruent
/// to `p - 1` modulo `p`. Since `p - 1` is the canonical representative of
/// `-1` modulo `p`, this is the classical statement `(p - 1)! ≡ -1 (mod p)`.
/// This is `prime_imp_wilson_factorial_congr` from wilson.ac, restated.
theorem wilsons_theorem(p: Nat) {
    p.is_prime implies (p - Nat.1).factorial.congr_mod(p - Nat.1, p)
} by {
    if p.is_prime {
        prime_imp_wilson_factorial_congr(p)
        (p - Nat.1).factorial.congr_mod(p - Nat.1, p)
    }
}

// ---------------------------------------------------------------------------
// The multiplicative order divides the totient.
// ---------------------------------------------------------------------------

/// The multiplicative order of `a` modulo `n` divides Euler's totient
/// `φ(n)`, whenever `a` is coprime to `n` and `n > 0`. This is
/// `multiplicative_order_mod_divides_totient` from multiplicative_order.ac,
/// restated.
theorem order_divides_totient(a: Nat, n: Nat) {
    n != Nat.0 and a.coprime(n)
        implies multiplicative_order_mod(a, n).divides(n.totient)
} by {
    if n != Nat.0 and a.coprime(n) {
        multiplicative_order_mod_divides_totient(a, n)
        multiplicative_order_mod(a, n).divides(n.totient)
    }
}

// ---------------------------------------------------------------------------
// Fermat's little theorem.
// ---------------------------------------------------------------------------

/// Fermat's little theorem for an arbitrary base: for a prime `p` and any
/// natural `a`, the power `a^p` is congruent to `a` modulo `p`. This holds
/// for every `a` without any coprimality hypothesis. This is
/// `fermats_little_congr` from fermat.ac, restated.
theorem fermats_little_all_a(p: Nat, a: Nat) {
    p.is_prime implies a.pow(p).congr_mod(a, p)
} by {
    if p.is_prime {
        fermats_little_congr(p, a)
        a.pow(p).congr_mod(a, p)
    }
}

/// Fermat's little theorem for base two: for a prime `p > 2`, the power
/// `2^(p - 1)` is congruent to `1` modulo `p`. An odd prime does not divide
/// `2`, so this is the unit case `fermat_euler` from fermat.ac applied at
/// `a = 2`.
theorem fermat_base_two(p: Nat) {
    p.is_prime and Nat.2 < p implies Nat.2.pow(p - Nat.1).congr_mod(Nat.1, p)
} by {
    if p.is_prime and Nat.2 < p {
        Nat.1 <= Nat.2
        Nat.2 < p
        coprime_below_prime(p, Nat.2)
        Nat.2.coprime(p)
        fermat_euler(p, Nat.2)
        Nat.2.pow(p - Nat.1).congr_mod(Nat.1, p)
    }
}
