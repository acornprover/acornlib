/// Euler's totient applications.
///
/// This file collects the classical applications of Euler's totient `φ(n)`
/// (the library's `n.totient` attribute from `number_theory.totient`) as
/// standalone statements: Euler's theorem, the parity of the totient, the
/// product formula (prime-power and two-prime cases), the counting
/// characterisation of the totient as the number of integers `1 <= k <= n`
/// coprime to `n`, and the prime-power formula `φ(p^k) = p^k - p^(k-1)`.
///
/// Every theorem here is proved in its home file — `totient.ac`,
/// `totient_sums.ac`, `totient_deep.ac`, `euler_products.ac` or
/// `fermat.ac` — and is restated here in one place with a citation to the
/// original proof.
from nat import Nat, pos_of_ne_zero
from number_theory.totient import euler, count_coprime_to, range_filter_coprime_length
from number_theory.totient_sums import count_gcd_eq, count_gcd_eq_le,
    count_gcd_eq_le_eq_to, count_gcd_eq_totient_quotient, totient_prime_power
from number_theory.totient_deep import is_even, lower_half_residues,
    totient_even, totient_even_double, totient_prime_power_product_form,
    totient_pq
from number_theory.dirichlet import divisor_quotient, divisor_quotient_one
from number_theory.fermat import fermat_euler
numerals Nat

// ---------------------------------------------------------------------------
// Euler's theorem.
// ---------------------------------------------------------------------------

/// Euler's theorem: for `n > 0` and `a` coprime to `n`, the power `a^φ(n)` is
/// congruent to `1` modulo `n`. This is the statement of `euler` from
/// `totient.ac`, restated here (also recorded as `euler_theorem_general` in
/// `modular_applications.ac` and `euler_theorem` in `modular_power.ac`).
theorem totient_euler_theorem(n: Nat, a: Nat) {
    n != Nat.0 and a.coprime(n) implies a.pow(n.totient).congr_mod(Nat.1, n)
} by {
    if n != Nat.0 and a.coprime(n) {
        euler(n, a)
        a.pow(n.totient).congr_mod(Nat.1, n)
    }
}

/// Euler's theorem at a prime modulus: for prime `p` and `a` coprime to `p`,
/// `a^(p - 1) ≡ 1 (mod p)`. Since `φ(p) = p - 1` (`totient_prime`), this is
/// Euler's theorem specialised to `n = p`, and is the classical
/// Fermat–Euler result; restated from `fermat_euler` in `fermat.ac`.
theorem totient_euler_theorem_prime(p: Nat, a: Nat) {
    p.is_prime and a.coprime(p) implies a.pow(p - Nat.1).congr_mod(Nat.1, p)
} by {
    if p.is_prime and a.coprime(p) {
        fermat_euler(p, a)
        a.pow(p - Nat.1).congr_mod(Nat.1, p)
    }
}

// ---------------------------------------------------------------------------
// The parity of the totient.
// ---------------------------------------------------------------------------

/// Euler's totient is even for `n > 2`: the reflection `a -> n - a` pairs the
/// reduced residues without fixed points, so they split into two halves of
/// equal size. Restates `totient_even` from `totient_deep.ac`.
theorem totient_even_for_n_gt_two(n: Nat) {
    Nat.2 < n implies is_even(n.totient)
} by {
    if Nat.2 < n {
        totient_even(n)
        is_even(n.totient)
    }
}

/// The explicit double form of the parity result: for `n > 2`, the totient is
/// twice the number of reduced residues in the lower half `(0, n/2)`.
/// Restates `totient_even_double` from `totient_deep.ac`.
theorem totient_even_double_form(n: Nat) {
    Nat.2 < n implies n.totient = Nat.2 * lower_half_residues(n).length
} by {
    if Nat.2 < n {
        totient_even_double(n)
        n.totient = Nat.2 * lower_half_residues(n).length
    }
}

// ---------------------------------------------------------------------------
// The product formula.
// ---------------------------------------------------------------------------

// The general product formula `φ(n) = n * prod_{p | n} (1 - 1/p)` is stated
// in this library in the natural-valued form
//
//   φ(n) = prod_{p | n} (p^(a_p - 1) * (p - 1)),   a_p = count_prime_factor(p, n),
//
// with the product over the distinct prime divisors of `n`.  The local factor
// at `p^k || n` is `p^(k-1) * (p - 1)` (the formula's `p^k * (1 - 1/p)`), and
// `φ` is multiplicative on coprime arguments, so the local factors multiply to
// `φ(n)`.  The prime-power and two-prime cases are proved below; the general
// statement needs the full multiplicativity of `φ` over the prime
// factorisation, which is left for the factorisation lane (see the comment in
// `totient_deep.ac`).

/// The totient product formula at a prime power: `φ(p^k) = p^(k-1) * (p - 1)`
/// for prime `p` and `k >= 1` — the natural-number reading of
/// `φ(p^k) = p^k * (1 - 1/p)`. Restates `totient_prime_power_product_form`
/// from `totient_deep.ac` (also `nat_totient_prime_pow_product_form` from
/// `euler_products.ac`).
theorem totient_product_formula_prime_power(p: Nat, k: Nat) {
    Nat.1 <= k and p.is_prime
        implies (p.pow(k)).totient = p.pow(k - Nat.1) * (p - Nat.1)
} by {
    if Nat.1 <= k and p.is_prime {
        totient_prime_power_product_form(p, k)
        (p.pow(k)).totient = p.pow(k - Nat.1) * (p - Nat.1)
    }
}

/// The totient product formula at a product of two distinct primes:
/// `φ(p * q) = (p - 1) * (q - 1)`, the factored form of
/// `φ(p * q) = p * q * (1 - 1/p) * (1 - 1/q)`. Restates
/// `totient_pq` from `totient_deep.ac` (also
/// `nat_totient_two_prime_product_form` from `euler_products.ac`).
theorem totient_product_formula_two_primes(p: Nat, q: Nat) {
    p.is_prime and q.is_prime and p != q
        implies (p * q).totient = (p - Nat.1) * (q - Nat.1)
} by {
    if p.is_prime and q.is_prime and p != q {
        totient_pq(p, q)
        (p * q).totient = (p - Nat.1) * (q - Nat.1)
    }
}

// ---------------------------------------------------------------------------
// The counting characterisation of the totient.
// ---------------------------------------------------------------------------

/// Euler's totient is the number of integers `1 <= k <= n` with `gcd(k, n) = 1`:
/// `count_gcd_eq_le(n, 1, n) = φ(n)`. The closed-range fiber counts of
/// `gcd(_, n)` agree with the open-range ones at the full bound
/// (`count_gcd_eq_le_eq_to`), and the `d = 1` fiber of the open range `[0, n)`
/// is `φ(n)` since `n / 1 = n` (`count_gcd_eq_totient_quotient` at `d = 1`
/// together with `divisor_quotient_one`).
theorem totient_counts_coprime_up_to(n: Nat) {
    count_gcd_eq_le(n, Nat.1, n) = n.totient
} by {
    if n = Nat.0 {
        n.totient = Nat.0.totient
        Nat.0.totient = count_coprime_to(Nat.0, Nat.0)
        count_coprime_to(Nat.0, Nat.0) = Nat.0
        n.totient = Nat.0
        count_gcd_eq_le(n, Nat.1, n) = count_gcd_eq_le(Nat.0, Nat.1, Nat.0)
        count_gcd_eq_le(Nat.0, Nat.1, Nat.0) = Nat.0
        count_gcd_eq_le(n, Nat.1, n) = Nat.0
        count_gcd_eq_le(n, Nat.1, n) = n.totient
    } else {
        n != Nat.0
        pos_of_ne_zero(n)
        Nat.0 < n
        count_gcd_eq_le_eq_to(n, Nat.1)
        count_gcd_eq_le(n, Nat.1, n) = count_gcd_eq(n, Nat.1, n)
        Nat.1 * n = n
        Nat.1.divides(n)
        count_gcd_eq_totient_quotient(n, Nat.1)
        count_gcd_eq(n, Nat.1, n) = (divisor_quotient(n, Nat.1)).totient
        divisor_quotient_one(n)
        divisor_quotient(n, Nat.1) = n
        (divisor_quotient(n, Nat.1)).totient = n.totient
        count_gcd_eq(n, Nat.1, n) = n.totient
        count_gcd_eq_le(n, Nat.1, n) = n.totient
    }
}

/// The open-range counting characterisation: filtering the range `[0, n)` by
/// coprimality to `n` gives a list of length `φ(n)`. Restates
/// `range_filter_coprime_length` from `totient.ac`.
theorem totient_coprime_range_filter_length(n: Nat) {
    n.range.filter(function(x: Nat) { x.coprime(n) }).length = n.totient
} by {
    range_filter_coprime_length(n)
    n.range.filter(function(x: Nat) { x.coprime(n) }).length = n.totient
}

// ---------------------------------------------------------------------------
// The prime-power formula.
// ---------------------------------------------------------------------------

/// Euler's totient at a prime power: `φ(p^k) = p^k - p^(k-1)` for prime `p`
/// and `k >= 1`. Directly, the numbers not coprime to `p^k` are exactly the
/// multiples of `p`, of which there are `p^(k-1)` below `p^k`. Restates
/// `totient_prime_power` from `totient_sums.ac` (also
/// `totient_prime_power_deep` in `totient_deep.ac`).
theorem totient_prime_power_formula(p: Nat, k: Nat) {
    Nat.1 <= k and p.is_prime
        implies (p.pow(k)).totient = p.pow(k) - p.pow(k - Nat.1)
} by {
    if Nat.1 <= k and p.is_prime {
        totient_prime_power(p, k)
        (p.pow(k)).totient = p.pow(k) - p.pow(k - Nat.1)
    }
}
