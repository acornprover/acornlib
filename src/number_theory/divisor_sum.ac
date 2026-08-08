from nat import Nat
from list import List, map
from list import sum, map_sum_add
from algebra.add_semigroup import add_fn
from nat import divides_self, divides_zero, divides_lte, lte_antisymm, lte_trans,
    add_zero_right, add_suc_right, not_lt_zero, add_suc_left, add_zero_left,
    add_one_right, lte_ref
from number_theory.factorisation import prime_divisor_is_one_or_self
from number_theory.arithmetic_functions import arithmetic_fn_add, arithmetic_fn_add_apply, nat_one_arithmetic_fn, nat_identity_arithmetic_fn, nat_zero_arithmetic_fn
numerals Nat

/// The list of divisors `d` of `n` with `0 < d <= k`, in descending order.
/// A helper that lets the divisor list be built by induction on the bound.
define divisors_up_to(n: Nat, k: Nat) -> List[Nat] {
    match k {
        Nat.zero {
            List.nil[Nat]
        }
        Nat.suc(j) {
            if k.divides(n) {
                List.cons(k, divisors_up_to(n, j))
            } else {
                divisors_up_to(n, j)
            }
        }
    }
}

/// The list of positive divisors of `n` up to `n` itself, in descending order.
/// For `n >= 1` this enumerates every positive divisor of `n`.
define divisor_list(n: Nat) -> List[Nat] {
    divisors_up_to(n, n)
}

/// Recurrence: stepping past a divisor index conses it onto the divisor list.
theorem divisors_up_to_suc_yes(n: Nat, k: Nat) {
    k.suc.divides(n)
        implies divisors_up_to(n, k.suc) =
            List.cons(k.suc, divisors_up_to(n, k))
}

/// Recurrence: stepping past a non-divisor index leaves the divisor list alone.
theorem divisors_up_to_suc_no(n: Nat, k: Nat) {
    not k.suc.divides(n)
        implies divisors_up_to(n, k.suc) = divisors_up_to(n, k)
}

/// Base case: there are no positive divisors at the empty bound.
theorem divisors_up_to_zero(n: Nat) {
    divisors_up_to(n, Nat.0) = List.nil[Nat]
}

/// The divisor-sum operator: `divisor_sum_fn(f)(n) = sum_{d | n, d <= n} f(d)`.
/// For `n >= 1` this is the usual `sum_{d | n} f(d)` from elementary number theory.
define divisor_sum_fn(f: Nat -> Nat) -> (Nat -> Nat) {
    function(n: Nat) { sum(map(divisor_list(n), f)) }
}

/// Application of the divisor-sum operator unfolds to a list sum over divisors.
theorem divisor_sum_fn_apply(f: Nat -> Nat, n: Nat) {
    divisor_sum_fn(f)(n) = sum(map(divisor_list(n), f))
}

/// The empty list is the divisor list of zero.
theorem divisor_list_zero {
    divisor_list(Nat.0) = List.nil[Nat]
} by {
    divisor_list(Nat.0) = divisors_up_to(Nat.0, Nat.0)
    divisors_up_to_zero(Nat.0)
}

/// The divisor list of one contains only one.
theorem divisor_list_one {
    divisor_list(Nat.1) = List.cons(Nat.1, List.nil[Nat])
} by {
    divides_self(Nat.1)
    Nat.1.divides(Nat.1)
    divisors_up_to_suc_yes(Nat.1, Nat.0)
    divisors_up_to(Nat.1, Nat.0.suc) = List.cons(Nat.0.suc, divisors_up_to(Nat.1, Nat.0))
    Nat.0.suc = Nat.1
    divisors_up_to_zero(Nat.1)
    divisors_up_to(Nat.1, Nat.0) = List.nil[Nat]
    divisors_up_to(Nat.1, Nat.1) = List.cons(Nat.1, List.nil[Nat])
    divisor_list(Nat.1) = divisors_up_to(Nat.1, Nat.1)
}

/// The number of positive divisors of `n` (the standard `tau` function).
define nat_tau(n: Nat) -> Nat {
    divisor_list(n).length
}

/// The sum of the positive divisors of `n` (the standard `sigma` function).
define nat_sigma(n: Nat) -> Nat {
    sum(divisor_list(n))
}

/// `tau(0) = 0`, since zero has no positive divisors below it.
theorem nat_tau_zero {
    nat_tau(Nat.0) = Nat.0
} by {
    divisor_list_zero
    nat_tau(Nat.0) = List.nil[Nat].length
}

/// `tau(1) = 1`, since one has exactly one positive divisor.
theorem nat_tau_one {
    nat_tau(Nat.1) = Nat.1
} by {
    divisor_list_one
    nat_tau(Nat.1) = List.cons(Nat.1, List.nil[Nat]).length
    List.cons(Nat.1, List.nil[Nat]).length = List.nil[Nat].length.suc
    List.nil[Nat].length = Nat.0
}

/// `sigma(0) = 0`, since the divisor list of zero is empty.
theorem nat_sigma_zero {
    nat_sigma(Nat.0) = Nat.0
} by {
    divisor_list_zero
    nat_sigma(Nat.0) = sum(List.nil[Nat])
}

/// `1` divides every natural number, since `1 * n = n`.
theorem one_divides_nat(n: Nat) {
    Nat.1.divides(n)
} by {
    Nat.1 * n = n
    Nat.1.divides(n) = exists(c: Nat) { Nat.1 * c = n }
}

/// At bound `1`, the divisor list is `[1]`.
theorem divisors_up_to_one(n: Nat) {
    divisors_up_to(n, Nat.1) = List.cons(Nat.1, List.nil[Nat])
} by {
    one_divides_nat(n)
    Nat.0.suc = Nat.1
    Nat.0.suc.divides(n)
    divisors_up_to_suc_yes(n, Nat.0)
    divisors_up_to(n, Nat.0.suc) =
        List.cons(Nat.0.suc, divisors_up_to(n, Nat.0))
    divisors_up_to_zero(n)
}

/// If `p` is prime and `1 < k < p`, then `k` does not divide `p`.
theorem prime_strict_below_not_divides(p: Nat, k: Nat) {
    p.is_prime and Nat.1 < k and k < p implies not k.divides(p)
} by {
    if p.is_prime and Nat.1 < k and k < p {
        k != Nat.1
        k != p
        if k.divides(p) {
            prime_divisor_is_one_or_self(p, k)
            false
        }
    }
}

/// Step lemma: if `p` is prime and `1 <= x` with `x.suc < p`, the bound step
/// does not introduce a new divisor.
theorem divisors_up_to_prime_step(p: Nat, x: Nat) {
    p.is_prime and Nat.1 <= x and x.suc < p
        implies divisors_up_to(p, x.suc) = divisors_up_to(p, x)
} by {
    if p.is_prime and Nat.1 <= x and x.suc < p {
        Nat.1 < x.suc
        prime_strict_below_not_divides(p, x.suc)
        not x.suc.divides(p)
        divisors_up_to_suc_no(p, x)
        divisors_up_to(p, x.suc) = divisors_up_to(p, x)
    }
}

/// Inductive predicate for the prime-offset divisor list lemma.
define divisors_up_to_prime_offset_pred(p: Nat, y: Nat) -> Bool {
    Nat.1 + y < p implies
        divisors_up_to(p, Nat.1 + y) = List.cons(Nat.1, List.nil[Nat])
}

/// Base case: at offset zero the divisor list is `[1]`.
theorem divisors_up_to_prime_offset_base(p: Nat) {
    divisors_up_to_prime_offset_pred(p, Nat.0)
} by {
    Nat.1 + Nat.0 = Nat.1
    divisors_up_to_one(p)
}

/// Step case: the offset induction continues across `y -> y.suc` for a prime `p`.
theorem divisors_up_to_prime_offset_step(p: Nat, y: Nat) {
    p.is_prime and divisors_up_to_prime_offset_pred(p, y)
        implies divisors_up_to_prime_offset_pred(p, y.suc)
} by {
    if p.is_prime and divisors_up_to_prime_offset_pred(p, y) {
        if Nat.1 + y.suc < p {
            Nat.1 + y.suc = (Nat.1 + y).suc
            Nat.1 + y < (Nat.1 + y).suc
            Nat.1 + y < p
            divisors_up_to(p, Nat.1 + y) = List.cons(Nat.1, List.nil[Nat])
            Nat.1 <= Nat.1 + y
            (Nat.1 + y).suc < p
            divisors_up_to_prime_step(p, Nat.1 + y)
            divisors_up_to(p, (Nat.1 + y).suc) = divisors_up_to(p, Nat.1 + y)
            divisors_up_to(p, Nat.1 + y.suc) = List.cons(Nat.1, List.nil[Nat])
        }
        divisors_up_to_prime_offset_pred(p, y.suc)
    }
}

/// Below `p`, induct on the offset `j`: at bound `Nat.1 + j` the divisor list
/// of a prime is `[1]` whenever `Nat.1 + j < p`.
theorem divisors_up_to_prime_offset(p: Nat, j: Nat) {
    p.is_prime implies divisors_up_to_prime_offset_pred(p, j)
} by {
    if p.is_prime {
        divisors_up_to_prime_offset_base(p)
        divisors_up_to_prime_offset_pred(p, Nat.0)
        forall(y: Nat) {
            if divisors_up_to_prime_offset_pred(p, y) {
                divisors_up_to_prime_offset_step(p, y)
                divisors_up_to_prime_offset_pred(p, y.suc)
            }
        }
        divisors_up_to_prime_offset_pred(p, j)
    }
}

/// Witness lemma: a prime is `r.suc.suc` for some `r`.
theorem prime_eq_suc_suc(p: Nat) {
    p.is_prime implies exists(r: Nat) { r.suc.suc = p }
} by {
    if p.is_prime {
        Nat.1 < p
        let q: Nat satisfy { q.suc = p }
        Nat.0 < q
        let r: Nat satisfy { r.suc = q }
        r.suc.suc = p
    }
}

/// Helper: at bound `r.suc` (which is `Nat.1 + r`), the bounded divisor list
/// of a prime `r.suc.suc` is `[1]`.
theorem divisors_up_to_prime_at_pred(r: Nat) {
    r.suc.suc.is_prime
        implies divisors_up_to(r.suc.suc, r.suc) = List.cons(Nat.1, List.nil[Nat])
} by {
    if r.suc.suc.is_prime {
        Nat.1 + r = r.suc
        r.suc < r.suc.suc
        Nat.1 + r < r.suc.suc
        divisors_up_to_prime_offset(r.suc.suc, r)
        divisors_up_to_prime_offset_pred(r.suc.suc, r)
        divisors_up_to(r.suc.suc, Nat.1 + r) = List.cons(Nat.1, List.nil[Nat])
        divisors_up_to(r.suc.suc, r.suc) = List.cons(Nat.1, List.nil[Nat])
    }
}

/// `sigma(1) = 1`, since the only positive divisor of one is one.
theorem nat_sigma_one {
    nat_sigma(Nat.1) = Nat.1
} by {
    divisor_list_one
    nat_sigma(Nat.1) = sum(List.cons(Nat.1, List.nil[Nat]))
}

/// Helper: at `r.suc.suc` prime, the bounded divisor list at bound `r.suc.suc` is `[r.suc.suc, 1]`.
theorem divisors_up_to_prime_at(r: Nat) {
    r.suc.suc.is_prime implies
        divisors_up_to(r.suc.suc, r.suc.suc) =
            List.cons(r.suc.suc, List.cons(Nat.1, List.nil[Nat]))
} by {
    if r.suc.suc.is_prime {
        divisors_up_to_prime_at_pred(r)
        divisors_up_to(r.suc.suc, r.suc) = List.cons(Nat.1, List.nil[Nat])
        divides_self(r.suc.suc)
        r.suc.suc.divides(r.suc.suc)
        divisors_up_to_suc_yes(r.suc.suc, r.suc)
        divisors_up_to(r.suc.suc, r.suc.suc) =
            List.cons(r.suc.suc, divisors_up_to(r.suc.suc, r.suc))
        List.cons(r.suc.suc, divisors_up_to(r.suc.suc, r.suc)) =
            List.cons(r.suc.suc, List.cons(Nat.1, List.nil[Nat]))
        divisors_up_to(r.suc.suc, r.suc.suc) =
            List.cons(r.suc.suc, List.cons(Nat.1, List.nil[Nat]))
    }
}

/// The divisor list of a prime `p` is `[p, 1]`.
theorem divisor_list_prime(p: Nat) {
    p.is_prime implies divisor_list(p) = List.cons(p, List.cons(Nat.1, List.nil[Nat]))
} by {
    if p.is_prime {
        prime_eq_suc_suc(p)
        let r: Nat satisfy { r.suc.suc = p }
        r.suc.suc.is_prime
        divisors_up_to_prime_at(r)
        divisors_up_to(r.suc.suc, r.suc.suc) =
            List.cons(r.suc.suc, List.cons(Nat.1, List.nil[Nat]))
        p = r.suc.suc
        divisors_up_to(p, p) =
            List.cons(r.suc.suc, List.cons(Nat.1, List.nil[Nat]))
        List.cons(r.suc.suc, List.cons(Nat.1, List.nil[Nat])) =
            List.cons(p, List.cons(Nat.1, List.nil[Nat]))
        divisors_up_to(p, p) = List.cons(p, List.cons(Nat.1, List.nil[Nat]))
        divisor_list(p) = divisors_up_to(p, p)
        divisor_list(p) = List.cons(p, List.cons(Nat.1, List.nil[Nat]))
    }
}

/// `tau(p) = 2` for primes `p`.
theorem nat_tau_prime(p: Nat) {
    p.is_prime implies nat_tau(p) = Nat.2
} by {
    if p.is_prime {
        divisor_list_prime(p)
        divisor_list(p) = List.cons(p, List.cons(Nat.1, List.nil[Nat]))
        nat_tau(p) = divisor_list(p).length
        nat_tau(p) = List.cons(p, List.cons(Nat.1, List.nil[Nat])).length
        List.cons(p, List.cons(Nat.1, List.nil[Nat])).length =
            List.cons(Nat.1, List.nil[Nat]).length.suc
        List.cons(Nat.1, List.nil[Nat]).length = List.nil[Nat].length.suc
        List.nil[Nat].length = Nat.0
        List.cons(Nat.1, List.nil[Nat]).length = Nat.0.suc
        List.cons(p, List.cons(Nat.1, List.nil[Nat])).length = Nat.0.suc.suc
        nat_tau(p) = Nat.0.suc.suc
        Nat.0.suc.suc = Nat.1.suc
        Nat.1.suc = Nat.2
        nat_tau(p) = Nat.2
    }
}

/// At zero argument the divisor sum is empty, so `divisor_sum_fn(f)(0) = 0`.
theorem divisor_sum_fn_at_zero(f: Nat -> Nat) {
    divisor_sum_fn(f)(Nat.0) = Nat.0
} by {
    divisor_list_zero
    divisor_list(Nat.0) = List.nil[Nat]
    map(List.nil[Nat], f) = List.nil[Nat]
    sum(List.nil[Nat]) = Nat.0
    divisor_sum_fn(f)(Nat.0) = sum(map(divisor_list(Nat.0), f))
}

/// At argument one the only divisor is one, so `divisor_sum_fn(f)(1) = f(1)`.
theorem divisor_sum_fn_at_one(f: Nat -> Nat) {
    divisor_sum_fn(f)(Nat.1) = f(Nat.1)
} by {
    divisor_list_one
    divisor_list(Nat.1) = List.cons(Nat.1, List.nil[Nat])
    map(List.cons(Nat.1, List.nil[Nat]), f) =
        List.cons(f(Nat.1), map(List.nil[Nat], f))
    map(List.nil[Nat], f) = List.nil[Nat]
    map(List.cons(Nat.1, List.nil[Nat]), f) = List.cons(f(Nat.1), List.nil[Nat])
    sum(List.cons(f(Nat.1), List.nil[Nat])) = f(Nat.1) + sum(List.nil[Nat])
    sum(List.nil[Nat]) = Nat.0
    f(Nat.1) + Nat.0 = f(Nat.1)
    divisor_sum_fn(f)(Nat.1) = sum(map(divisor_list(Nat.1), f))
}

/// At a prime argument `p`, the divisor sum collects the values at `p` and at
/// `1`: `divisor_sum_fn(f)(p) = f(p) + f(1)`.
theorem divisor_sum_fn_at_prime(f: Nat -> Nat, p: Nat) {
    p.is_prime implies divisor_sum_fn(f)(p) = f(p) + f(Nat.1)
} by {
    if p.is_prime {
        divisor_list_prime(p)
        divisor_list(p) = List.cons(p, List.cons(Nat.1, List.nil[Nat]))
        map(List.cons(p, List.cons(Nat.1, List.nil[Nat])), f) =
            List.cons(f(p), map(List.cons(Nat.1, List.nil[Nat]), f))
        map(List.cons(Nat.1, List.nil[Nat]), f) =
            List.cons(f(Nat.1), map(List.nil[Nat], f))
        map(List.nil[Nat], f) = List.nil[Nat]
        map(List.cons(p, List.cons(Nat.1, List.nil[Nat])), f) =
            List.cons(f(p), List.cons(f(Nat.1), List.nil[Nat]))
        sum(List.nil[Nat]) = Nat.0
        sum(List.cons(f(Nat.1), List.nil[Nat])) =
            f(Nat.1) + sum(List.nil[Nat])
        sum(List.cons(f(Nat.1), List.nil[Nat])) = f(Nat.1) + Nat.0
        f(Nat.1) + Nat.0 = f(Nat.1)
        sum(List.cons(f(Nat.1), List.nil[Nat])) = f(Nat.1)
        sum(List.cons(f(p), List.cons(f(Nat.1), List.nil[Nat]))) =
            f(p) + sum(List.cons(f(Nat.1), List.nil[Nat]))
        sum(List.cons(f(p), List.cons(f(Nat.1), List.nil[Nat]))) =
            f(p) + f(Nat.1)
        divisor_sum_fn(f)(p) = sum(map(divisor_list(p), f))
        sum(map(divisor_list(p), f)) =
            sum(map(List.cons(p, List.cons(Nat.1, List.nil[Nat])), f))
        sum(map(List.cons(p, List.cons(Nat.1, List.nil[Nat])), f)) =
            sum(List.cons(f(p), List.cons(f(Nat.1), List.nil[Nat])))
        divisor_sum_fn(f)(p) = f(p) + f(Nat.1)
    }
}

/// The pointwise-sum and curried `add_fn` views agree as functions.
theorem arithmetic_fn_add_eq_add_fn(f: Nat -> Nat, g: Nat -> Nat) {
    arithmetic_fn_add(f, g) = add_fn[Nat, Nat](f, g)
} by {
    forall(n: Nat) {
        arithmetic_fn_add_apply(f, g, n)
        arithmetic_fn_add(f, g)(n) = f(n) + g(n)
        add_fn[Nat, Nat](f, g, n) = f(n) + g(n)
        arithmetic_fn_add(f, g)(n) = add_fn[Nat, Nat](f, g, n)
    }
}

/// The divisor-sum operator is additive in its argument.
theorem divisor_sum_fn_add(f: Nat -> Nat, g: Nat -> Nat, n: Nat) {
    divisor_sum_fn(arithmetic_fn_add(f, g))(n) =
        divisor_sum_fn(f)(n) + divisor_sum_fn(g)(n)
} by {
    let l: List[Nat] = divisor_list(n)
    map_sum_add[Nat, Nat](l, f, g)
    sum(map(l, f)) + sum(map(l, g)) = sum(map(l, add_fn[Nat, Nat](f, g)))
    arithmetic_fn_add_eq_add_fn(f, g)
    arithmetic_fn_add(f, g) = add_fn[Nat, Nat](f, g)
    sum(map(l, add_fn[Nat, Nat](f, g))) = sum(map(l, arithmetic_fn_add(f, g)))
    divisor_sum_fn(f)(n) = sum(map(divisor_list(n), f))
    divisor_sum_fn(g)(n) = sum(map(divisor_list(n), g))
    divisor_sum_fn(arithmetic_fn_add(f, g))(n) =
        sum(map(divisor_list(n), arithmetic_fn_add(f, g)))
    divisor_sum_fn(f)(n) + divisor_sum_fn(g)(n) =
        sum(map(l, f)) + sum(map(l, g))
}

/// Summing the constant-one arithmetic function over a list gives the list length.
theorem sum_map_nat_one_arithmetic_fn_eq_length(l: List[Nat]) {
    sum(map(l, nat_one_arithmetic_fn)) = l.length
} by {
    define p(xs: List[Nat]) -> Bool {
        sum(map(xs, nat_one_arithmetic_fn)) = xs.length
    }
    map(List.nil[Nat], nat_one_arithmetic_fn) = List.nil[Nat]
    sum(List.nil[Nat]) = Nat.0
    List.nil[Nat].length = Nat.0
    p(List.nil)
    forall(head: Nat, tail: List[Nat]) {
        if p(tail) {
            map(List.cons(head, tail), nat_one_arithmetic_fn) =
                List.cons(nat_one_arithmetic_fn(head), map(tail, nat_one_arithmetic_fn))
            nat_one_arithmetic_fn(head) = Nat.1
            map(List.cons(head, tail), nat_one_arithmetic_fn) =
                List.cons(Nat.1, map(tail, nat_one_arithmetic_fn))
            sum(List.cons(Nat.1, map(tail, nat_one_arithmetic_fn))) =
                Nat.1 + sum(map(tail, nat_one_arithmetic_fn))
            sum(map(List.cons(head, tail), nat_one_arithmetic_fn)) =
                Nat.1 + sum(map(tail, nat_one_arithmetic_fn))
            sum(map(tail, nat_one_arithmetic_fn)) = tail.length
            Nat.1 + tail.length = tail.length.suc
            List.cons(head, tail).length = tail.length.suc
            sum(map(List.cons(head, tail), nat_one_arithmetic_fn)) =
                List.cons(head, tail).length
            p(List.cons(head, tail))
        }
    }
    p(l)
}

/// The divisor sum of the constant-one arithmetic function is `tau`.
theorem divisor_sum_fn_nat_one_arithmetic_fn_eq_tau(n: Nat) {
    divisor_sum_fn(nat_one_arithmetic_fn)(n) = nat_tau(n)
} by {
    let l: List[Nat] = divisor_list(n)
    divisor_sum_fn(nat_one_arithmetic_fn)(n) =
        sum(map(divisor_list(n), nat_one_arithmetic_fn))
    sum_map_nat_one_arithmetic_fn_eq_length(l)
    sum(map(l, nat_one_arithmetic_fn)) = l.length
    nat_tau(n) = divisor_list(n).length
}

/// Summing the identity arithmetic function over a list gives the list sum.
theorem sum_map_nat_identity_arithmetic_fn_eq_sum(l: List[Nat]) {
    sum(map(l, nat_identity_arithmetic_fn)) = sum(l)
} by {
    define p(xs: List[Nat]) -> Bool {
        sum(map(xs, nat_identity_arithmetic_fn)) = sum(xs)
    }
    map(List.nil[Nat], nat_identity_arithmetic_fn) = List.nil[Nat]
    p(List.nil)
    forall(head: Nat, tail: List[Nat]) {
        if p(tail) {
            map(List.cons(head, tail), nat_identity_arithmetic_fn) =
                List.cons(nat_identity_arithmetic_fn(head), map(tail, nat_identity_arithmetic_fn))
            nat_identity_arithmetic_fn(head) = head
            map(List.cons(head, tail), nat_identity_arithmetic_fn) =
                List.cons(head, map(tail, nat_identity_arithmetic_fn))
            sum(List.cons(head, map(tail, nat_identity_arithmetic_fn))) =
                head + sum(map(tail, nat_identity_arithmetic_fn))
            sum(map(List.cons(head, tail), nat_identity_arithmetic_fn)) =
                head + sum(map(tail, nat_identity_arithmetic_fn))
            sum(map(tail, nat_identity_arithmetic_fn)) = sum(tail)
            sum(List.cons(head, tail)) = head + sum(tail)
            sum(map(List.cons(head, tail), nat_identity_arithmetic_fn)) =
                sum(List.cons(head, tail))
            p(List.cons(head, tail))
        }
    }
    p(l)
}

/// Summing the constant-zero arithmetic function over a list gives zero.
theorem sum_map_nat_zero_arithmetic_fn_eq_zero(l: List[Nat]) {
    sum(map(l, nat_zero_arithmetic_fn)) = Nat.0
} by {
    define p(xs: List[Nat]) -> Bool {
        sum(map(xs, nat_zero_arithmetic_fn)) = Nat.0
    }
    map(List.nil[Nat], nat_zero_arithmetic_fn) = List.nil[Nat]
    sum(List.nil[Nat]) = Nat.0
    p(List.nil)
    forall(head: Nat, tail: List[Nat]) {
        if p(tail) {
            map(List.cons(head, tail), nat_zero_arithmetic_fn) =
                List.cons(nat_zero_arithmetic_fn(head), map(tail, nat_zero_arithmetic_fn))
            nat_zero_arithmetic_fn(head) = Nat.0
            map(List.cons(head, tail), nat_zero_arithmetic_fn) =
                List.cons(Nat.0, map(tail, nat_zero_arithmetic_fn))
            sum(List.cons(Nat.0, map(tail, nat_zero_arithmetic_fn))) =
                Nat.0 + sum(map(tail, nat_zero_arithmetic_fn))
            sum(map(tail, nat_zero_arithmetic_fn)) = Nat.0
            Nat.0 + Nat.0 = Nat.0
            sum(map(List.cons(head, tail), nat_zero_arithmetic_fn)) = Nat.0
            p(List.cons(head, tail))
        }
    }
    p(l)
}

/// The divisor sum of the constant-zero arithmetic function is itself.
theorem divisor_sum_fn_nat_zero_arithmetic_fn(n: Nat) {
    divisor_sum_fn(nat_zero_arithmetic_fn)(n) = nat_zero_arithmetic_fn(n)
} by {
    let l: List[Nat] = divisor_list(n)
    divisor_sum_fn(nat_zero_arithmetic_fn)(n) =
        sum(map(divisor_list(n), nat_zero_arithmetic_fn))
    sum_map_nat_zero_arithmetic_fn_eq_zero(l)
    sum(map(l, nat_zero_arithmetic_fn)) = Nat.0
    nat_zero_arithmetic_fn(n) = Nat.0
}

/// The divisor sum of the identity arithmetic function is `sigma`.
theorem divisor_sum_fn_nat_identity_arithmetic_fn_eq_sigma(n: Nat) {
    divisor_sum_fn(nat_identity_arithmetic_fn)(n) = nat_sigma(n)
} by {
    let l: List[Nat] = divisor_list(n)
    divisor_sum_fn(nat_identity_arithmetic_fn)(n) =
        sum(map(divisor_list(n), nat_identity_arithmetic_fn))
    sum_map_nat_identity_arithmetic_fn_eq_sum(l)
    sum(map(l, nat_identity_arithmetic_fn)) = sum(l)
    nat_sigma(n) = sum(divisor_list(n))
}

/// `sigma(p) = p + 1` for primes `p`.
theorem nat_sigma_prime(p: Nat) {
    p.is_prime implies nat_sigma(p) = p + Nat.1
} by {
    if p.is_prime {
        divisor_list_prime(p)
        divisor_list(p) = List.cons(p, List.cons(Nat.1, List.nil[Nat]))
        nat_sigma(p) = sum(divisor_list(p))
        nat_sigma(p) = sum(List.cons(p, List.cons(Nat.1, List.nil[Nat])))
        sum(List.nil[Nat]) = Nat.0
        sum(List.cons(Nat.1, List.nil[Nat])) = Nat.1 + sum(List.nil[Nat])
        sum(List.cons(Nat.1, List.nil[Nat])) = Nat.1 + Nat.0
        sum(List.cons(p, List.cons(Nat.1, List.nil[Nat]))) =
            p + sum(List.cons(Nat.1, List.nil[Nat]))
        sum(List.cons(p, List.cons(Nat.1, List.nil[Nat]))) = p + (Nat.1 + Nat.0)
        Nat.1 + Nat.0 = Nat.1
        p + (Nat.1 + Nat.0) = p + Nat.1
        nat_sigma(p) = p + Nat.1
    }
}

/// At every bound `k`, every entry of `divisors_up_to(n, k)` lies in the
/// range `1..=k` and divides `n`.
theorem divisors_up_to_member(n: Nat, k: Nat, d: Nat) {
    divisors_up_to(n, k).contains(d) implies
        (Nat.0 < d and d <= k and d.divides(n))
} by {
    define p(x: Nat) -> Bool {
        divisors_up_to(n, x).contains(d) implies
            (Nat.0 < d and d <= x and d.divides(n))
    }
    divisors_up_to_zero(n)
    divisors_up_to(n, Nat.0) = List.nil[Nat]
    not List.nil[Nat].contains(d)
    if divisors_up_to(n, Nat.0).contains(d) {
        false
    }
    p(Nat.0)
    forall(j: Nat) {
        if p(j) {
            if divisors_up_to(n, j.suc).contains(d) {
                if j.suc.divides(n) {
                    divisors_up_to_suc_yes(n, j)
                    divisors_up_to(n, j.suc) = List.cons(j.suc, divisors_up_to(n, j))
                    List.cons(j.suc, divisors_up_to(n, j)).contains(d)
                    if d = j.suc {
                        Nat.0 < j.suc
                        Nat.0 < d
                        d <= j.suc
                        d.divides(n)
                        Nat.0 < d and d <= j.suc and d.divides(n)
                    } else {
                        d != j.suc
                        divisors_up_to(n, j).contains(d)
                        Nat.0 < d
                        d <= j
                        j <= j.suc
                        d <= j.suc
                        d.divides(n)
                        Nat.0 < d and d <= j.suc and d.divides(n)
                    }
                    Nat.0 < d and d <= j.suc and d.divides(n)
                } else {
                    not j.suc.divides(n)
                    divisors_up_to_suc_no(n, j)
                    divisors_up_to(n, j.suc) = divisors_up_to(n, j)
                    divisors_up_to(n, j).contains(d)
                    Nat.0 < d
                    d <= j
                    j <= j.suc
                    d <= j.suc
                    d.divides(n)
                    Nat.0 < d and d <= j.suc and d.divides(n)
                }
                Nat.0 < d and d <= j.suc and d.divides(n)
            }
            p(j.suc)
        }
    }
    p(k)
}

/// Every entry of `divisor_list(n)` is a positive divisor of `n`.
theorem divisor_list_contains_implies(n: Nat, d: Nat) {
    divisor_list(n).contains(d) implies (Nat.0 < d and d.divides(n))
} by {
    if divisor_list(n).contains(d) {
        divisors_up_to(n, n).contains(d)
        divisors_up_to_member(n, n, d)
        Nat.0 < d and d <= n and d.divides(n)
        Nat.0 < d
        d.divides(n)
    }
}

/// Converse direction: a positive divisor of `n` at most `k` lies in
/// `divisors_up_to(n, k)`.
theorem divisors_up_to_complete(n: Nat, k: Nat, d: Nat) {
    Nat.0 < d and d <= k and d.divides(n)
        implies divisors_up_to(n, k).contains(d)
} by {
    define p(x: Nat) -> Bool {
        Nat.0 < d and d <= x and d.divides(n)
            implies divisors_up_to(n, x).contains(d)
    }
    if Nat.0 < d and d <= Nat.0 and d.divides(n) {
        false
    }
    p(Nat.0)
    forall(j: Nat) {
        if p(j) {
            if Nat.0 < d and d <= j.suc and d.divides(n) {
                if d = j.suc {
                    j.suc.divides(n)
                    divisors_up_to_suc_yes(n, j)
                    divisors_up_to(n, j.suc) = List.cons(j.suc, divisors_up_to(n, j))
                    List.cons(j.suc, divisors_up_to(n, j)).contains(j.suc)
                    divisors_up_to(n, j.suc).contains(d)
                } else {
                    d != j.suc
                    d <= j
                    divisors_up_to(n, j).contains(d)
                    if j.suc.divides(n) {
                        divisors_up_to_suc_yes(n, j)
                        divisors_up_to(n, j.suc) = List.cons(j.suc, divisors_up_to(n, j))
                        List.cons(j.suc, divisors_up_to(n, j)).contains(d)
                        divisors_up_to(n, j.suc).contains(d)
                    } else {
                        divisors_up_to_suc_no(n, j)
                        divisors_up_to(n, j.suc) = divisors_up_to(n, j)
                        divisors_up_to(n, j.suc).contains(d)
                    }
                }
            }
            p(j.suc)
        }
    }
    p(k)
}

/// A positive divisor `d` of positive `n` appears in `divisor_list(n)`.
theorem divisor_list_contains_of(n: Nat, d: Nat) {
    Nat.0 < n and d.divides(n) and Nat.0 < d
        implies divisor_list(n).contains(d)
} by {
    if Nat.0 < n and d.divides(n) and Nat.0 < d {
        divides_lte(d, n)
        d <= n
        divisors_up_to_complete(n, n, d)
        divisors_up_to(n, n).contains(d)
    }
}

/// Inductive predicate for uniqueness of `divisors_up_to`.
define divisors_up_to_unique_pred(n: Nat, k: Nat) -> Bool {
    divisors_up_to(n, k).is_unique
}

/// Base case for uniqueness: the empty list is unique.
theorem divisors_up_to_unique_base(n: Nat) {
    divisors_up_to_unique_pred(n, Nat.0)
} by {
    divisors_up_to_zero(n)
    divisors_up_to(n, Nat.0) = List.nil[Nat]
    List.nil[Nat].unique = List.nil[Nat]
    List.nil[Nat].is_unique
}

/// Step case for uniqueness.
theorem divisors_up_to_unique_step(n: Nat, k: Nat) {
    divisors_up_to_unique_pred(n, k)
        implies divisors_up_to_unique_pred(n, k.suc)
} by {
    if divisors_up_to_unique_pred(n, k) {
        divisors_up_to(n, k).is_unique
        divisors_up_to(n, k).unique = divisors_up_to(n, k)
        if k.suc.divides(n) {
            divisors_up_to_suc_yes(n, k)
            divisors_up_to(n, k.suc) = List.cons(k.suc, divisors_up_to(n, k))
            if divisors_up_to(n, k).contains(k.suc) {
                divisors_up_to_member(n, k, k.suc)
                k.suc <= k
                k < k.suc
                false
            }
            not divisors_up_to(n, k).contains(k.suc)
            List.cons(k.suc, divisors_up_to(n, k)).unique =
                List.cons(k.suc, divisors_up_to(n, k).unique)
            List.cons(k.suc, divisors_up_to(n, k)).unique =
                List.cons(k.suc, divisors_up_to(n, k))
            List.cons(k.suc, divisors_up_to(n, k)).is_unique
            divisors_up_to(n, k.suc).is_unique
        } else {
            divisors_up_to_suc_no(n, k)
            divisors_up_to(n, k.suc) = divisors_up_to(n, k)
            divisors_up_to(n, k.suc).is_unique
        }
        divisors_up_to_unique_pred(n, k.suc)
    }
}

/// At every bound `k`, `divisors_up_to(n, k)` is a unique list.
theorem divisors_up_to_unique(n: Nat, k: Nat) {
    divisors_up_to_unique_pred(n, k)
} by {
    define p(x: Nat) -> Bool { divisors_up_to_unique_pred(n, x) }
    divisors_up_to_unique_base(n)
    p(Nat.0)
    forall(j: Nat) {
        if p(j) {
            divisors_up_to_unique_pred(n, j)
            divisors_up_to_unique_step(n, j)
            divisors_up_to_unique_pred(n, j.suc)
            p(j.suc)
        }
    }
    p(k)
}

/// The divisor list of `n` is unique.
theorem divisor_list_is_unique(n: Nat) {
    divisor_list(n).is_unique
} by {
    divisors_up_to_unique(n, n)
    divisors_up_to_unique_pred(n, n)
    divisors_up_to(n, n).is_unique
    divisor_list(n) = divisors_up_to(n, n)
}
