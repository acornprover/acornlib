from nat import Nat
from nat import exp_zero, exp_one, exp_add, div_mul, lte_add_left, add_imp_sub,
    add_assoc, add_comm
from list import List, map, sum, product, partial, partial_split_last, partial_one,
    unique_same_contains_map_sum_eq, map_map, sum_map_of_pointwise
from data.nat.nat_squarefree import is_squarefree, one_is_squarefree
from number_theory.squarefree import prime_is_squarefree, prime_pow_not_squarefree,
    product_of_unique_primes_squarefree
from number_theory.divisor_sum import divisor_list, divisor_sum_fn, divisor_sum_fn_apply,
    divisor_sum_fn_at_prime, divisor_list_is_unique
from number_theory.sigma_multiplicative import divisor_list_prime_pow_contains_iff,
    pow_range_list_unique
from number_theory.liouville import divisor_list_six
from number_theory.mobius_inversion import nat_mobius, nat_mobius_one, nat_mobius_prime,
    nat_mobius_multiplicative_apply
from number_theory.carmichael import two_is_prime, three_is_prime, three_ne_two
from number_theory.totient import totient_pq
from number_theory.factorisation import all_prime, all_prime_nil, all_prime_cons_intro,
    coprime_of_distinct_primes
from list import singleton_unique, singleton_contains_imp_eq,
    cons_unique_of_tail_unique_not_contains
from int import Int
from data.basic.functions import compose
numerals Nat
numerals Int

// ---------------------------------------------------------------------------
// Small concrete natural-number facts
// ---------------------------------------------------------------------------

/// Two plus `m` is `m` with two successors.
theorem two_add_suc_suc(m: Nat) {
    Nat.2 + m = m.suc.suc
} by {
    Nat.1 + Nat.1 = Nat.2
    Nat.1 + m = m.suc
    Nat.1 + m.suc = m.suc.suc
    Nat.1 + Nat.1 + m = Nat.1 + (Nat.1 + m)
    Nat.1 + (Nat.1 + m) = Nat.1 + m.suc
    Nat.1 + (Nat.1 + m) = m.suc.suc
    Nat.2 + m = Nat.1 + Nat.1 + m
    Nat.2 + m = m.suc.suc
}

/// Three plus six is nine.
theorem nat_add_3_6 {
    Nat.3 + Nat.6 = Nat.9
} by {
    Nat.1 + Nat.2 = Nat.3
    Nat.3 + Nat.6 = (Nat.1 + Nat.2) + Nat.6
    add_assoc(Nat.1, Nat.2, Nat.6)
    (Nat.1 + Nat.2) + Nat.6 = Nat.1 + (Nat.2 + Nat.6)
    two_add_suc_suc(Nat.6)
    Nat.2 + Nat.6 = Nat.6.suc.suc
    Nat.6.suc.suc = Nat.8
    Nat.2 + Nat.6 = Nat.8
    Nat.1 + (Nat.2 + Nat.6) = Nat.1 + Nat.8
    Nat.1 + Nat.8 = Nat.8.suc
    Nat.8.suc = Nat.9
    Nat.1 + Nat.8 = Nat.9
    Nat.1 + (Nat.2 + Nat.6) = Nat.9
    (Nat.1 + Nat.2) + Nat.6 = Nat.9
    Nat.3 + Nat.6 = Nat.9
}

/// The sum `1 + (2 + (3 + 6))` is twelve.
theorem six_sum_value {
    Nat.1 + (Nat.2 + (Nat.3 + Nat.6)) = Nat.12
} by {
    nat_add_3_6
    Nat.3 + Nat.6 = Nat.9
    Nat.2 + (Nat.3 + Nat.6) = Nat.2 + Nat.9
    two_add_suc_suc(Nat.9)
    Nat.2 + Nat.9 = Nat.9.suc.suc
    Nat.9.suc.suc = Nat.11
    Nat.2 + Nat.9 = Nat.11
    Nat.2 + (Nat.3 + Nat.6) = Nat.11
    Nat.1 + (Nat.2 + (Nat.3 + Nat.6)) = Nat.1 + Nat.11
    Nat.1 + Nat.11 = Nat.11.suc
    Nat.11.suc = Nat.12
    Nat.1 + Nat.11 = Nat.12
    Nat.1 + (Nat.2 + (Nat.3 + Nat.6)) = Nat.12
}

/// Seven minus five is two.
theorem seven_minus_five {
    Nat.7 - Nat.5 = Nat.2
} by {
    Nat.2 + Nat.5 = Nat.7
    add_imp_sub(Nat.2, Nat.5, Nat.7)
    Nat.2 = Nat.7 - Nat.5
    Nat.7 - Nat.5 = Nat.2
}

/// The sum `0 + (2 + 3)` is five.
theorem neg_sum_value {
    Nat.0 + (Nat.2 + Nat.3) = Nat.5
} by {
    two_add_suc_suc(Nat.3)
    Nat.2 + Nat.3 = Nat.3.suc.suc
    Nat.3.suc.suc = Nat.5
    Nat.2 + Nat.3 = Nat.5
    Nat.0 + (Nat.2 + Nat.3) = Nat.0 + Nat.5
    Nat.0 + Nat.5 = Nat.5
    Nat.0 + (Nat.2 + Nat.3) = Nat.5
}

/// The sum `1 + 6` is seven.
theorem pos_sum_value {
    Nat.1 + Nat.6 = Nat.7
} by {
    Nat.1 + Nat.6 = Nat.6.suc
    Nat.6.suc = Nat.7
    Nat.1 + Nat.6 = Nat.7
}

// ---------------------------------------------------------------------------
// Dedekind's psi function
// ---------------------------------------------------------------------------

/// The summand of the squarefree-divisor sum defining `dedekind_psi`: the
/// complementary divisor `n / d` at squarefree divisors `d` of `n`, and `0`
/// elsewhere.
define psi_summand(n: Nat) -> (Nat -> Nat) {
    function(d: Nat) { if is_squarefree(d) { n.div(d) } else { Nat.0 } }
}

/// Dedekind's psi function `psi(n) = n * prod_{p | n} (1 + 1/p)`.
///
/// Over the naturals the product form is expanded by
/// `prod_{p | n} (1 + 1/p) = sum_{d | n, d squarefree} 1 / d`, so the
/// definition is the squarefree-divisor sum
/// `psi(n) = sum_{d | n, d squarefree} n / d`.
define dedekind_psi(n: Nat) -> Nat {
    divisor_sum_fn(psi_summand(n))(n)
}

/// Dedekind's psi function as an arithmetic function.
let nat_dedekind_psi: Nat -> Nat = function(n: Nat) { dedekind_psi(n) }

/// Dedekind psi at a prime: `psi(p) = p + 1`, since the only prime divisor of
/// `p` is `p` itself and `p * (1 + 1 / p) = p + 1`.
theorem dedekind_psi_prime(p: Nat) {
    p.is_prime implies dedekind_psi(p) = p + Nat.1
} by {
    if p.is_prime {
        dedekind_psi(p) = divisor_sum_fn(psi_summand(p))(p)
        divisor_sum_fn_at_prime(psi_summand(p), p)
        divisor_sum_fn(psi_summand(p))(p) = psi_summand(p)(p) + psi_summand(p)(Nat.1)
        prime_is_squarefree(p)
        is_squarefree(p)
        psi_summand(p)(p) = p.div(p)
        Nat.1 < p
        p != Nat.0
        div_mul(Nat.1, p)
        (Nat.1 * p).div(p) = Nat.1
        Nat.1 * p = p
        p.div(p) = Nat.1
        psi_summand(p)(p) = Nat.1
        one_is_squarefree
        is_squarefree(Nat.1)
        psi_summand(p)(Nat.1) = p.div(Nat.1)
        div_mul(p, Nat.1)
        (p * Nat.1).div(Nat.1) = p
        p * Nat.1 = p
        p.div(Nat.1) = p
        psi_summand(p)(Nat.1) = p
        divisor_sum_fn(psi_summand(p))(p) = Nat.1 + p
        Nat.1 + p = p + Nat.1
        divisor_sum_fn(psi_summand(p))(p) = p + Nat.1
        dedekind_psi(p) = p + Nat.1
    }
}

/// A partial sum whose terms vanish from index two onward collapses to its
/// first two terms: `partial(g, n + 2) = g(0) + g(1)`.
theorem partial_two_term(g: Nat -> Nat) {
    (forall(i: Nat) { Nat.2 <= i implies g(i) = Nat.0 })
        implies forall(n: Nat) { partial(g, n.suc.suc) = g(Nat.0) + g(Nat.1) }
} by {
    if forall(i: Nat) { Nat.2 <= i implies g(i) = Nat.0 } {
        let f: Nat -> Bool = function(x: Nat) {
            partial(g, x.suc.suc) = g(Nat.0) + g(Nat.1)
        }
        partial_split_last(g, Nat.1)
        partial(g, Nat.1.suc) = partial(g, Nat.1) + g(Nat.1)
        partial_one(g)
        partial(g, Nat.1) = g(Nat.0)
        Nat.1.suc = Nat.2
        partial(g, Nat.2) = g(Nat.0) + g(Nat.1)
        Nat.0.suc.suc = Nat.2
        partial(g, Nat.0.suc.suc) = g(Nat.0) + g(Nat.1)
        f(Nat.0)
        forall(x: Nat) {
            if f(x) {
                partial(g, x.suc.suc) = g(Nat.0) + g(Nat.1)
                partial_split_last(g, x.suc.suc)
                partial(g, x.suc.suc.suc) = partial(g, x.suc.suc) + g(x.suc.suc)
                lte_add_left(Nat.2, Nat.0, x)
                Nat.2 + Nat.0 <= Nat.2 + x
                Nat.2 + Nat.0 = Nat.2
                Nat.2 <= Nat.2 + x
                two_add_suc_suc(x)
                Nat.2 + x = x.suc.suc
                Nat.2 <= x.suc.suc
                forall(i: Nat) { Nat.2 <= i implies g(i) = Nat.0 }
                Nat.2 <= x.suc.suc implies g(x.suc.suc) = Nat.0
                g(x.suc.suc) = Nat.0
                (g(Nat.0) + g(Nat.1)) + Nat.0 = g(Nat.0) + g(Nat.1)
                partial(g, x.suc.suc.suc) = g(Nat.0) + g(Nat.1)
                (x.suc).suc.suc = x.suc.suc.suc
                partial(g, (x.suc).suc.suc) = g(Nat.0) + g(Nat.1)
                f(x.suc)
            }
        }
        f(Nat.0) and forall(x: Nat) { f(x) implies f(x.suc) }
        Nat.induction(f)
        forall(n: Nat) {
            f(n)
            partial(g, n.suc.suc) = g(Nat.0) + g(Nat.1)
        }
    }
}

/// The `dedekind_psi` summand evaluated along the powers of a prime:
/// `g(i) = psi(p^(n+1))`-summand at `p^i`.
define psi_pow_summand(p: Nat, n: Nat) -> (Nat -> Nat) {
    function(i: Nat) { psi_summand(p.pow(n.suc))(p.pow(i)) }
}

/// The prime-power summand vanishes from the square of the prime onward:
/// `g(i) = 0` for `i >= 2`, since `p^i` is not squarefree.
theorem psi_pow_summand_vanishes(p: Nat, n: Nat) {
    p.is_prime implies forall(i: Nat) {
        Nat.2 <= i implies psi_pow_summand(p, n)(i) = Nat.0
    }
} by {
    if p.is_prime {
        forall(i: Nat) {
            if Nat.2 <= i {
                prime_pow_not_squarefree(p, i)
                not is_squarefree(p.pow(i))
                psi_summand(p.pow(n.suc))(p.pow(i)) = Nat.0
                psi_pow_summand(p, n)(i) = psi_summand(p.pow(n.suc))(p.pow(i))
                psi_pow_summand(p, n)(i) = Nat.0
            }
        }
    }
}

/// Dedekind psi at a prime power: `psi(p^(n+1)) = p^(n+1) + p^n`, i.e.
/// `psi(p^k) = p^k + p^(k-1)` for `k = n + 1`, since `p^k * (1 + 1/p)` is
/// `p^k + p^(k-1)`.  The only squarefree divisors of `p^(n+1)` are `1` and
/// `p`, so the squarefree-divisor sum has exactly two nonzero terms.
theorem dedekind_psi_prime_pow(p: Nat, n: Nat) {
    p.is_prime implies dedekind_psi(p.pow(n.suc)) = p.pow(n.suc) + p.pow(n)
} by {
    if p.is_prime {
        dedekind_psi(p.pow(n.suc)) = divisor_sum_fn(psi_summand(p.pow(n.suc)))(p.pow(n.suc))
        divisor_sum_fn_apply(psi_summand(p.pow(n.suc)), p.pow(n.suc))
        divisor_sum_fn(psi_summand(p.pow(n.suc)))(p.pow(n.suc)) =
            sum(map(divisor_list(p.pow(n.suc)), psi_summand(p.pow(n.suc))))
        divisor_list_is_unique(p.pow(n.suc))
        divisor_list(p.pow(n.suc)).is_unique
        pow_range_list_unique(p, n.suc)
        map((n.suc + Nat.1).range, p.pow).is_unique
        forall(x: Nat) {
            divisor_list_prime_pow_contains_iff(p, n.suc, x)
            divisor_list(p.pow(n.suc)).contains(x) =
                map((n.suc + Nat.1).range, p.pow).contains(x)
        }
        unique_same_contains_map_sum_eq(divisor_list(p.pow(n.suc)),
            map((n.suc + Nat.1).range, p.pow), psi_summand(p.pow(n.suc)))
        sum(map(divisor_list(p.pow(n.suc)), psi_summand(p.pow(n.suc)))) =
            sum(map(map((n.suc + Nat.1).range, p.pow), psi_summand(p.pow(n.suc))))
        map_map((n.suc + Nat.1).range, p.pow, psi_summand(p.pow(n.suc)))
        map(map((n.suc + Nat.1).range, p.pow), psi_summand(p.pow(n.suc))) =
            map((n.suc + Nat.1).range, compose(psi_summand(p.pow(n.suc)), p.pow))
        sum(map(map((n.suc + Nat.1).range, p.pow), psi_summand(p.pow(n.suc)))) =
            sum(map((n.suc + Nat.1).range, compose(psi_summand(p.pow(n.suc)), p.pow)))
        n.suc + Nat.1 = n.suc.suc
        map((n.suc + Nat.1).range, compose(psi_summand(p.pow(n.suc)), p.pow)) =
            map(n.suc.suc.range, compose(psi_summand(p.pow(n.suc)), p.pow))
        sum(map((n.suc + Nat.1).range, compose(psi_summand(p.pow(n.suc)), p.pow))) =
            sum(map(n.suc.suc.range, compose(psi_summand(p.pow(n.suc)), p.pow)))
        sum(map(divisor_list(p.pow(n.suc)), psi_summand(p.pow(n.suc)))) =
            sum(map(n.suc.suc.range, compose(psi_summand(p.pow(n.suc)), p.pow)))
        // The composed map agrees pointwise with the named summand.
        forall(x: Nat) {
            if n.suc.suc.range.contains(x) {
                compose(psi_summand(p.pow(n.suc)), p.pow, x) =
                    psi_summand(p.pow(n.suc))(p.pow(x))
                psi_pow_summand(p, n)(x) = psi_summand(p.pow(n.suc))(p.pow(x))
                compose(psi_summand(p.pow(n.suc)), p.pow, x) =
                    psi_pow_summand(p, n)(x)
            }
        }
        sum_map_of_pointwise(n.suc.suc.range,
            compose(psi_summand(p.pow(n.suc)), p.pow), psi_pow_summand(p, n))
        sum(map(n.suc.suc.range, compose(psi_summand(p.pow(n.suc)), p.pow))) =
            sum(map(n.suc.suc.range, psi_pow_summand(p, n)))
        sum(map(divisor_list(p.pow(n.suc)), psi_summand(p.pow(n.suc)))) =
            sum(map(n.suc.suc.range, psi_pow_summand(p, n)))
        partial(psi_pow_summand(p, n), n.suc.suc) =
            sum(map(n.suc.suc.range, psi_pow_summand(p, n)))
        sum(map(divisor_list(p.pow(n.suc)), psi_summand(p.pow(n.suc)))) =
            partial(psi_pow_summand(p, n), n.suc.suc)
        // The partial sum collapses to its first two terms.
        psi_pow_summand_vanishes(p, n)
        partial_two_term(psi_pow_summand(p, n))
        partial(psi_pow_summand(p, n), n.suc.suc) =
            psi_pow_summand(p, n)(Nat.0) + psi_pow_summand(p, n)(Nat.1)
        // First term: at `p^0 = 1` the summand is `p^(n+1) / 1 = p^(n+1)`.
        exp_zero(p)
        p.pow(Nat.0) = Nat.1
        one_is_squarefree
        is_squarefree(Nat.1)
        psi_summand(p.pow(n.suc))(Nat.1) = p.pow(n.suc).div(Nat.1)
        psi_pow_summand(p, n)(Nat.0) = p.pow(n.suc).div(Nat.1)
        Nat.1 != Nat.0
        div_mul(p.pow(n.suc), Nat.1)
        (p.pow(n.suc) * Nat.1).div(Nat.1) = p.pow(n.suc)
        p.pow(n.suc) * Nat.1 = p.pow(n.suc)
        p.pow(n.suc).div(Nat.1) = p.pow(n.suc)
        psi_pow_summand(p, n)(Nat.0) = p.pow(n.suc)
        // Second term: at `p^1 = p` the summand is `p^(n+1) / p = p^n`.
        exp_one(p)
        p.pow(Nat.1) = p
        prime_is_squarefree(p)
        is_squarefree(p)
        psi_summand(p.pow(n.suc))(p) = p.pow(n.suc).div(p)
        psi_pow_summand(p, n)(Nat.1) = p.pow(n.suc).div(p)
        exp_add(p, n, Nat.1)
        p.pow(n + Nat.1) = p.pow(n) * p.pow(Nat.1)
        n + Nat.1 = n.suc
        p.pow(n.suc) = p.pow(n) * p.pow(Nat.1)
        exp_one(p)
        p.pow(Nat.1) = p
        p.pow(n.suc) = p.pow(n) * p
        Nat.1 < p
        p != Nat.0
        div_mul(p.pow(n), p)
        (p.pow(n) * p).div(p) = p.pow(n)
        p.pow(n.suc).div(p) = p.pow(n)
        psi_pow_summand(p, n)(Nat.1) = p.pow(n)
        // Assemble the two nonzero terms.
        partial(psi_pow_summand(p, n), n.suc.suc) =
            p.pow(n.suc) + p.pow(n)
        dedekind_psi(p.pow(n.suc)) = p.pow(n.suc) + p.pow(n)
    }
}

/// Two and three are the prime factors of six: `[2, 3]` is an all-prime list.
theorem all_prime_two_three {
    all_prime(List.cons(Nat.2, List.cons(Nat.3, List.nil[Nat])))
} by {
    all_prime_nil
    all_prime(List.nil[Nat])
    three_is_prime
    Nat.3.is_prime
    all_prime_cons_intro(Nat.3, List.nil[Nat])
    all_prime(List.cons(Nat.3, List.nil[Nat]))
    two_is_prime
    Nat.2.is_prime
    all_prime_cons_intro(Nat.2, List.cons(Nat.3, List.nil[Nat]))
    all_prime(List.cons(Nat.2, List.cons(Nat.3, List.nil[Nat])))
}

/// The list `[2, 3]` has no repetitions.
theorem two_three_unique {
    List.cons(Nat.2, List.cons(Nat.3, List.nil[Nat])).is_unique
} by {
    singleton_unique(Nat.3)
    List.cons(Nat.3, List.nil[Nat]).is_unique
    three_ne_two
    Nat.3 != Nat.2
    if List.cons(Nat.3, List.nil[Nat]).contains(Nat.2) {
        singleton_contains_imp_eq(Nat.3, Nat.2)
        Nat.2 = Nat.3
        Nat.3 != Nat.2
        false
    }
    not List.cons(Nat.3, List.nil[Nat]).contains(Nat.2)
    cons_unique_of_tail_unique_not_contains(Nat.2, List.cons(Nat.3, List.nil[Nat]))
    List.cons(Nat.2, List.cons(Nat.3, List.nil[Nat])).is_unique
}

/// Six is squarefree, since it is the product of the distinct primes two and
/// three.
theorem six_is_squarefree {
    is_squarefree(Nat.6)
} by {
    all_prime_two_three
    two_three_unique
    product_of_unique_primes_squarefree(List.cons(Nat.2, List.cons(Nat.3, List.nil[Nat])))
    is_squarefree(product[Nat](List.cons(Nat.2, List.cons(Nat.3, List.nil[Nat]))))
    product[Nat](List.cons(Nat.2, List.cons(Nat.3, List.nil[Nat]))) =
        Nat.2 * product[Nat](List.cons(Nat.3, List.nil[Nat]))
    product[Nat](List.cons(Nat.3, List.nil[Nat])) = Nat.3 * product[Nat](List.nil[Nat])
    product[Nat](List.nil[Nat]) = Nat.1
    Nat.3 * Nat.1 = Nat.3
    product[Nat](List.cons(Nat.3, List.nil[Nat])) = Nat.3
    Nat.2 * Nat.3 = Nat.6
    product[Nat](List.cons(Nat.2, List.cons(Nat.3, List.nil[Nat]))) = Nat.6
    is_squarefree(Nat.6)
}

/// Dedekind psi at six: `psi(6) = 6 * (1 + 1/2) * (1 + 1/3) = 12`.
theorem dedekind_psi_six {
    dedekind_psi(Nat.6) = Nat.12
} by {
    dedekind_psi(Nat.6) = divisor_sum_fn(psi_summand(Nat.6))(Nat.6)
    divisor_sum_fn_apply(psi_summand(Nat.6), Nat.6)
    divisor_sum_fn(psi_summand(Nat.6))(Nat.6) =
        sum(map(divisor_list(Nat.6), psi_summand(Nat.6)))
    divisor_list_six
    divisor_list(Nat.6) =
        List.cons(Nat.6, List.cons(Nat.3, List.cons(Nat.2, List.cons(Nat.1, List.nil[Nat]))))
    // The summands at the divisors 6, 3, 2, 1.
    six_is_squarefree
    is_squarefree(Nat.6)
    psi_summand(Nat.6)(Nat.6) = Nat.6.div(Nat.6)
    Nat.6 != Nat.0
    div_mul(Nat.1, Nat.6)
    (Nat.1 * Nat.6).div(Nat.6) = Nat.1
    Nat.1 * Nat.6 = Nat.6
    Nat.6.div(Nat.6) = Nat.1
    psi_summand(Nat.6)(Nat.6) = Nat.1
    three_is_prime
    Nat.3.is_prime
    prime_is_squarefree(Nat.3)
    is_squarefree(Nat.3)
    psi_summand(Nat.6)(Nat.3) = Nat.6.div(Nat.3)
    Nat.3 != Nat.0
    div_mul(Nat.2, Nat.3)
    (Nat.2 * Nat.3).div(Nat.3) = Nat.2
    Nat.2 * Nat.3 = Nat.6
    Nat.6.div(Nat.3) = Nat.2
    psi_summand(Nat.6)(Nat.3) = Nat.2
    two_is_prime
    Nat.2.is_prime
    prime_is_squarefree(Nat.2)
    is_squarefree(Nat.2)
    psi_summand(Nat.6)(Nat.2) = Nat.6.div(Nat.2)
    Nat.2 != Nat.0
    div_mul(Nat.3, Nat.2)
    (Nat.3 * Nat.2).div(Nat.2) = Nat.3
    Nat.3 * Nat.2 = Nat.6
    Nat.6.div(Nat.2) = Nat.3
    psi_summand(Nat.6)(Nat.2) = Nat.3
    one_is_squarefree
    is_squarefree(Nat.1)
    psi_summand(Nat.6)(Nat.1) = Nat.6.div(Nat.1)
    Nat.1 != Nat.0
    div_mul(Nat.6, Nat.1)
    (Nat.6 * Nat.1).div(Nat.1) = Nat.6
    Nat.6 * Nat.1 = Nat.6
    Nat.6.div(Nat.1) = Nat.6
    psi_summand(Nat.6)(Nat.1) = Nat.6
    // The mapped sum over the divisor list.
    map(List.cons(Nat.6, List.cons(Nat.3, List.cons(Nat.2, List.cons(Nat.1, List.nil[Nat])))),
        psi_summand(Nat.6)) =
        List.cons(psi_summand(Nat.6)(Nat.6),
            map(List.cons(Nat.3, List.cons(Nat.2, List.cons(Nat.1, List.nil[Nat]))),
                psi_summand(Nat.6)))
    map(List.cons(Nat.3, List.cons(Nat.2, List.cons(Nat.1, List.nil[Nat]))),
        psi_summand(Nat.6)) =
        List.cons(psi_summand(Nat.6)(Nat.3),
            map(List.cons(Nat.2, List.cons(Nat.1, List.nil[Nat])), psi_summand(Nat.6)))
    map(List.cons(Nat.2, List.cons(Nat.1, List.nil[Nat])), psi_summand(Nat.6)) =
        List.cons(psi_summand(Nat.6)(Nat.2),
            map(List.cons(Nat.1, List.nil[Nat]), psi_summand(Nat.6)))
    map(List.cons(Nat.1, List.nil[Nat]), psi_summand(Nat.6)) =
        List.cons(psi_summand(Nat.6)(Nat.1), map(List.nil[Nat], psi_summand(Nat.6)))
    map(List.nil[Nat], psi_summand(Nat.6)) = List.nil[Nat]
    map(List.cons(Nat.6, List.cons(Nat.3, List.cons(Nat.2, List.cons(Nat.1, List.nil[Nat])))),
        psi_summand(Nat.6)) =
        List.cons(Nat.1, List.cons(Nat.2, List.cons(Nat.3, List.cons(Nat.6, List.nil[Nat]))))
    sum(List.cons(Nat.1, List.cons(Nat.2, List.cons(Nat.3, List.cons(Nat.6, List.nil[Nat]))))) =
        Nat.1 + sum(List.cons(Nat.2, List.cons(Nat.3, List.cons(Nat.6, List.nil[Nat]))))
    sum(List.cons(Nat.2, List.cons(Nat.3, List.cons(Nat.6, List.nil[Nat])))) =
        Nat.2 + sum(List.cons(Nat.3, List.cons(Nat.6, List.nil[Nat])))
    sum(List.cons(Nat.3, List.cons(Nat.6, List.nil[Nat]))) =
        Nat.3 + sum(List.cons(Nat.6, List.nil[Nat]))
    sum(List.cons(Nat.6, List.nil[Nat])) = Nat.6 + sum(List.nil[Nat])
    sum(List.nil[Nat]) = Nat.0
    sum(List.cons(Nat.6, List.nil[Nat])) = Nat.6
    sum(List.cons(Nat.3, List.cons(Nat.6, List.nil[Nat]))) = Nat.3 + Nat.6
    sum(List.cons(Nat.2, List.cons(Nat.3, List.cons(Nat.6, List.nil[Nat])))) =
        Nat.2 + (Nat.3 + Nat.6)
    sum(List.cons(Nat.1, List.cons(Nat.2, List.cons(Nat.3, List.cons(Nat.6, List.nil[Nat]))))) =
        Nat.1 + (Nat.2 + (Nat.3 + Nat.6))
    six_sum_value
    Nat.1 + (Nat.2 + (Nat.3 + Nat.6)) = Nat.12
    sum(map(divisor_list(Nat.6), psi_summand(Nat.6))) = Nat.12
    dedekind_psi(Nat.6) = Nat.12
}

// ---------------------------------------------------------------------------
// The Jordan totient
// ---------------------------------------------------------------------------

/// The summand of the positive part of the Jordan totient: `(n/d)^k` at
/// divisors with `mu(d) = 1`, and `0` elsewhere.
define jordan_summand_pos(k: Nat, n: Nat) -> (Nat -> Nat) {
    function(d: Nat) { if nat_mobius(d) = Int.1 { (n.div(d)).pow(k) } else { Nat.0 } }
}

/// The summand of the negative part of the Jordan totient: `(n/d)^k` at
/// divisors with `mu(d) = -1`, and `0` elsewhere.
define jordan_summand_neg(k: Nat, n: Nat) -> (Nat -> Nat) {
    function(d: Nat) { if nat_mobius(d) = -Int.1 { (n.div(d)).pow(k) } else { Nat.0 } }
}

/// The positive part of the Jordan totient: `sum_{d | n, mu(d) = 1} (n/d)^k`.
define jordan_pos(k: Nat, n: Nat) -> Nat {
    divisor_sum_fn(jordan_summand_pos(k, n))(n)
}

/// The negative part of the Jordan totient: `sum_{d | n, mu(d) = -1} (n/d)^k`.
define jordan_neg(k: Nat, n: Nat) -> Nat {
    divisor_sum_fn(jordan_summand_neg(k, n))(n)
}

/// The Jordan totient `J_k(n) = n^k * prod_{p | n} (1 - 1/p^k)`, generalizing
/// Euler's totient (which is the case `k = 1`).
///
/// Expanding the product gives the Möbius form
/// `J_k(n) = sum_{d | n} mu(d) * (n / d)^k`.  Over the naturals the summands
/// with `mu(d) = 1` and `mu(d) = -1` are collected separately, and the value
/// is their difference; `mu` vanishes off the squarefree divisors, so both
/// sums are finite over the divisors of `n`.
define jordan_totient(k: Nat, n: Nat) -> Nat {
    jordan_pos(k, n) - jordan_neg(k, n)
}

/// The positive Jordan summand at a divisor with `mu(d) = 1`.
theorem jordan_summand_pos_at_one(k: Nat, n: Nat, d: Nat) {
    nat_mobius(d) = Int.1 implies jordan_summand_pos(k, n)(d) = (n.div(d)).pow(k)
}

/// The positive Jordan summand vanishes at divisors with `mu(d) != 1`.
theorem jordan_summand_pos_off_one(k: Nat, n: Nat, d: Nat) {
    nat_mobius(d) != Int.1 implies jordan_summand_pos(k, n)(d) = Nat.0
}

/// The negative Jordan summand at a divisor with `mu(d) = -1`.
theorem jordan_summand_neg_at_minus_one(k: Nat, n: Nat, d: Nat) {
    nat_mobius(d) = -Int.1 implies jordan_summand_neg(k, n)(d) = (n.div(d)).pow(k)
}

/// The negative Jordan summand vanishes at divisors with `mu(d) != -1`.
theorem jordan_summand_neg_off_minus_one(k: Nat, n: Nat, d: Nat) {
    nat_mobius(d) != -Int.1 implies jordan_summand_neg(k, n)(d) = Nat.0
}

/// The Möbius value of six is one: `mu(6) = mu(2) * mu(3) = 1`.
theorem mobius_six {
    nat_mobius(Nat.6) = Int.1
} by {
    two_is_prime
    three_is_prime
    three_ne_two
    Nat.3 != Nat.2
    Nat.2 != Nat.3
    coprime_of_distinct_primes(Nat.2, Nat.3)
    Nat.2.coprime(Nat.3)
    nat_mobius_multiplicative_apply(Nat.2, Nat.3)
    nat_mobius(Nat.2 * Nat.3) = nat_mobius(Nat.2) * nat_mobius(Nat.3)
    Nat.2 * Nat.3 = Nat.6
    nat_mobius(Nat.6) = nat_mobius(Nat.2) * nat_mobius(Nat.3)
    nat_mobius_prime(Nat.2)
    nat_mobius(Nat.2) = -Int.1
    nat_mobius_prime(Nat.3)
    nat_mobius(Nat.3) = -Int.1
    nat_mobius(Nat.2) * nat_mobius(Nat.3) = (-Int.1) * (-Int.1)
    (-Int.1) * (-Int.1) = Int.1
    nat_mobius(Nat.2) * nat_mobius(Nat.3) = Int.1
    nat_mobius(Nat.6) = Int.1
}

/// The Jordan totient at `k = 1` and `n = 6`: `J_1(6) = 2`, since
/// `6 * (1 - 1/2) * (1 - 1/3) = 2`.
theorem jordan_totient_six {
    jordan_totient(Nat.1, Nat.6) = Nat.2
} by {
    jordan_totient(Nat.1, Nat.6) = jordan_pos(Nat.1, Nat.6) - jordan_neg(Nat.1, Nat.6)
    // Positive part: mu(1) = mu(6) = 1 contribute 6/1 and 6/6.
    jordan_pos(Nat.1, Nat.6) = divisor_sum_fn(jordan_summand_pos(Nat.1, Nat.6))(Nat.6)
    divisor_sum_fn_apply(jordan_summand_pos(Nat.1, Nat.6), Nat.6)
    divisor_sum_fn(jordan_summand_pos(Nat.1, Nat.6))(Nat.6) =
        sum(map(divisor_list(Nat.6), jordan_summand_pos(Nat.1, Nat.6)))
    divisor_list_six
    divisor_list(Nat.6) =
        List.cons(Nat.6, List.cons(Nat.3, List.cons(Nat.2, List.cons(Nat.1, List.nil[Nat]))))
    mobius_six
    nat_mobius(Nat.6) = Int.1
    jordan_summand_pos_at_one(Nat.1, Nat.6, Nat.6)
    jordan_summand_pos(Nat.1, Nat.6)(Nat.6) = (Nat.6.div(Nat.6)).pow(Nat.1)
    Nat.6 != Nat.0
    div_mul(Nat.1, Nat.6)
    (Nat.1 * Nat.6).div(Nat.6) = Nat.1
    Nat.1 * Nat.6 = Nat.6
    Nat.6.div(Nat.6) = Nat.1
    exp_one(Nat.1)
    Nat.1.pow(Nat.1) = Nat.1
    jordan_summand_pos(Nat.1, Nat.6)(Nat.6) = Nat.1
    nat_mobius_prime(Nat.3)
    nat_mobius(Nat.3) = -Int.1
    -Int.1 != Int.1
    nat_mobius(Nat.3) != Int.1
    jordan_summand_pos_off_one(Nat.1, Nat.6, Nat.3)
    jordan_summand_pos(Nat.1, Nat.6)(Nat.3) = Nat.0
    nat_mobius_prime(Nat.2)
    nat_mobius(Nat.2) = -Int.1
    -Int.1 != Int.1
    nat_mobius(Nat.2) != Int.1
    jordan_summand_pos_off_one(Nat.1, Nat.6, Nat.2)
    jordan_summand_pos(Nat.1, Nat.6)(Nat.2) = Nat.0
    nat_mobius_one
    nat_mobius(Nat.1) = Int.1
    jordan_summand_pos_at_one(Nat.1, Nat.6, Nat.1)
    jordan_summand_pos(Nat.1, Nat.6)(Nat.1) = (Nat.6.div(Nat.1)).pow(Nat.1)
    Nat.1 != Nat.0
    div_mul(Nat.6, Nat.1)
    (Nat.6 * Nat.1).div(Nat.1) = Nat.6
    Nat.6 * Nat.1 = Nat.6
    Nat.6.div(Nat.1) = Nat.6
    exp_one(Nat.6)
    Nat.6.pow(Nat.1) = Nat.6
    jordan_summand_pos(Nat.1, Nat.6)(Nat.1) = Nat.6
    // The mapped sum over the divisor list for the positive part.
    map(List.cons(Nat.6, List.cons(Nat.3, List.cons(Nat.2, List.cons(Nat.1, List.nil[Nat])))),
        jordan_summand_pos(Nat.1, Nat.6)) =
        List.cons(jordan_summand_pos(Nat.1, Nat.6)(Nat.6),
            map(List.cons(Nat.3, List.cons(Nat.2, List.cons(Nat.1, List.nil[Nat]))),
                jordan_summand_pos(Nat.1, Nat.6)))
    map(List.cons(Nat.3, List.cons(Nat.2, List.cons(Nat.1, List.nil[Nat]))),
        jordan_summand_pos(Nat.1, Nat.6)) =
        List.cons(jordan_summand_pos(Nat.1, Nat.6)(Nat.3),
            map(List.cons(Nat.2, List.cons(Nat.1, List.nil[Nat])),
                jordan_summand_pos(Nat.1, Nat.6)))
    map(List.cons(Nat.2, List.cons(Nat.1, List.nil[Nat])),
        jordan_summand_pos(Nat.1, Nat.6)) =
        List.cons(jordan_summand_pos(Nat.1, Nat.6)(Nat.2),
            map(List.cons(Nat.1, List.nil[Nat]), jordan_summand_pos(Nat.1, Nat.6)))
    map(List.cons(Nat.1, List.nil[Nat]), jordan_summand_pos(Nat.1, Nat.6)) =
        List.cons(jordan_summand_pos(Nat.1, Nat.6)(Nat.1),
            map(List.nil[Nat], jordan_summand_pos(Nat.1, Nat.6)))
    map(List.nil[Nat], jordan_summand_pos(Nat.1, Nat.6)) = List.nil[Nat]
    map(List.cons(Nat.6, List.cons(Nat.3, List.cons(Nat.2, List.cons(Nat.1, List.nil[Nat])))),
        jordan_summand_pos(Nat.1, Nat.6)) =
        List.cons(Nat.1, List.cons(Nat.0, List.cons(Nat.0, List.cons(Nat.6, List.nil[Nat]))))
    sum(List.cons(Nat.1, List.cons(Nat.0, List.cons(Nat.0, List.cons(Nat.6, List.nil[Nat]))))) =
        Nat.1 + sum(List.cons(Nat.0, List.cons(Nat.0, List.cons(Nat.6, List.nil[Nat]))))
    sum(List.cons(Nat.0, List.cons(Nat.0, List.cons(Nat.6, List.nil[Nat])))) =
        Nat.0 + sum(List.cons(Nat.0, List.cons(Nat.6, List.nil[Nat])))
    sum(List.cons(Nat.0, List.cons(Nat.6, List.nil[Nat]))) =
        Nat.0 + sum(List.cons(Nat.6, List.nil[Nat]))
    sum(List.cons(Nat.6, List.nil[Nat])) = Nat.6 + sum(List.nil[Nat])
    sum(List.nil[Nat]) = Nat.0
    sum(List.cons(Nat.6, List.nil[Nat])) = Nat.6
    sum(List.cons(Nat.0, List.cons(Nat.6, List.nil[Nat]))) = Nat.0 + Nat.6
    Nat.0 + Nat.6 = Nat.6
    sum(List.cons(Nat.0, List.cons(Nat.6, List.nil[Nat]))) = Nat.6
    sum(List.cons(Nat.0, List.cons(Nat.0, List.cons(Nat.6, List.nil[Nat])))) =
        Nat.0 + Nat.6
    sum(List.cons(Nat.0, List.cons(Nat.0, List.cons(Nat.6, List.nil[Nat])))) = Nat.6
    sum(List.cons(Nat.1, List.cons(Nat.0, List.cons(Nat.0, List.cons(Nat.6, List.nil[Nat]))))) =
        Nat.1 + Nat.6
    pos_sum_value
    Nat.1 + Nat.6 = Nat.7
    sum(map(divisor_list(Nat.6), jordan_summand_pos(Nat.1, Nat.6))) = Nat.7
    jordan_pos(Nat.1, Nat.6) = Nat.7
    // Negative part: mu(3) = mu(2) = -1 contribute 6/3 and 6/2.
    jordan_neg(Nat.1, Nat.6) = divisor_sum_fn(jordan_summand_neg(Nat.1, Nat.6))(Nat.6)
    divisor_sum_fn_apply(jordan_summand_neg(Nat.1, Nat.6), Nat.6)
    divisor_sum_fn(jordan_summand_neg(Nat.1, Nat.6))(Nat.6) =
        sum(map(divisor_list(Nat.6), jordan_summand_neg(Nat.1, Nat.6)))
    divisor_list_six
    divisor_list(Nat.6) =
        List.cons(Nat.6, List.cons(Nat.3, List.cons(Nat.2, List.cons(Nat.1, List.nil[Nat]))))
    mobius_six
    nat_mobius(Nat.6) = Int.1
    Int.1 != -Int.1
    nat_mobius(Nat.6) != -Int.1
    jordan_summand_neg_off_minus_one(Nat.1, Nat.6, Nat.6)
    jordan_summand_neg(Nat.1, Nat.6)(Nat.6) = Nat.0
    nat_mobius_prime(Nat.3)
    nat_mobius(Nat.3) = -Int.1
    jordan_summand_neg_at_minus_one(Nat.1, Nat.6, Nat.3)
    jordan_summand_neg(Nat.1, Nat.6)(Nat.3) = (Nat.6.div(Nat.3)).pow(Nat.1)
    Nat.3 != Nat.0
    div_mul(Nat.2, Nat.3)
    (Nat.2 * Nat.3).div(Nat.3) = Nat.2
    Nat.2 * Nat.3 = Nat.6
    Nat.6.div(Nat.3) = Nat.2
    exp_one(Nat.2)
    Nat.2.pow(Nat.1) = Nat.2
    jordan_summand_neg(Nat.1, Nat.6)(Nat.3) = Nat.2
    nat_mobius_prime(Nat.2)
    nat_mobius(Nat.2) = -Int.1
    jordan_summand_neg_at_minus_one(Nat.1, Nat.6, Nat.2)
    jordan_summand_neg(Nat.1, Nat.6)(Nat.2) = (Nat.6.div(Nat.2)).pow(Nat.1)
    Nat.2 != Nat.0
    div_mul(Nat.3, Nat.2)
    (Nat.3 * Nat.2).div(Nat.2) = Nat.3
    Nat.3 * Nat.2 = Nat.6
    Nat.6.div(Nat.2) = Nat.3
    exp_one(Nat.3)
    Nat.3.pow(Nat.1) = Nat.3
    jordan_summand_neg(Nat.1, Nat.6)(Nat.2) = Nat.3
    nat_mobius_one
    nat_mobius(Nat.1) = Int.1
    Int.1 != -Int.1
    nat_mobius(Nat.1) != -Int.1
    jordan_summand_neg_off_minus_one(Nat.1, Nat.6, Nat.1)
    jordan_summand_neg(Nat.1, Nat.6)(Nat.1) = Nat.0
    // The mapped sum over the divisor list for the negative part.
    map(List.cons(Nat.6, List.cons(Nat.3, List.cons(Nat.2, List.cons(Nat.1, List.nil[Nat])))),
        jordan_summand_neg(Nat.1, Nat.6)) =
        List.cons(jordan_summand_neg(Nat.1, Nat.6)(Nat.6),
            map(List.cons(Nat.3, List.cons(Nat.2, List.cons(Nat.1, List.nil[Nat]))),
                jordan_summand_neg(Nat.1, Nat.6)))
    map(List.cons(Nat.3, List.cons(Nat.2, List.cons(Nat.1, List.nil[Nat]))),
        jordan_summand_neg(Nat.1, Nat.6)) =
        List.cons(jordan_summand_neg(Nat.1, Nat.6)(Nat.3),
            map(List.cons(Nat.2, List.cons(Nat.1, List.nil[Nat])),
                jordan_summand_neg(Nat.1, Nat.6)))
    map(List.cons(Nat.2, List.cons(Nat.1, List.nil[Nat])),
        jordan_summand_neg(Nat.1, Nat.6)) =
        List.cons(jordan_summand_neg(Nat.1, Nat.6)(Nat.2),
            map(List.cons(Nat.1, List.nil[Nat]), jordan_summand_neg(Nat.1, Nat.6)))
    map(List.cons(Nat.1, List.nil[Nat]), jordan_summand_neg(Nat.1, Nat.6)) =
        List.cons(jordan_summand_neg(Nat.1, Nat.6)(Nat.1),
            map(List.nil[Nat], jordan_summand_neg(Nat.1, Nat.6)))
    map(List.nil[Nat], jordan_summand_neg(Nat.1, Nat.6)) = List.nil[Nat]
    map(List.cons(Nat.6, List.cons(Nat.3, List.cons(Nat.2, List.cons(Nat.1, List.nil[Nat])))),
        jordan_summand_neg(Nat.1, Nat.6)) =
        List.cons(Nat.0, List.cons(Nat.2, List.cons(Nat.3, List.cons(Nat.0, List.nil[Nat]))))
    sum(List.cons(Nat.0, List.cons(Nat.2, List.cons(Nat.3, List.cons(Nat.0, List.nil[Nat]))))) =
        Nat.0 + sum(List.cons(Nat.2, List.cons(Nat.3, List.cons(Nat.0, List.nil[Nat]))))
    sum(List.cons(Nat.2, List.cons(Nat.3, List.cons(Nat.0, List.nil[Nat])))) =
        Nat.2 + sum(List.cons(Nat.3, List.cons(Nat.0, List.nil[Nat])))
    sum(List.cons(Nat.3, List.cons(Nat.0, List.nil[Nat]))) =
        Nat.3 + sum(List.cons(Nat.0, List.nil[Nat]))
    sum(List.cons(Nat.0, List.nil[Nat])) = Nat.0 + sum(List.nil[Nat])
    sum(List.nil[Nat]) = Nat.0
    sum(List.cons(Nat.0, List.nil[Nat])) = Nat.0 + Nat.0
    Nat.0 + Nat.0 = Nat.0
    sum(List.cons(Nat.0, List.nil[Nat])) = Nat.0
    sum(List.cons(Nat.3, List.cons(Nat.0, List.nil[Nat]))) = Nat.3 + Nat.0
    Nat.3 + Nat.0 = Nat.3
    sum(List.cons(Nat.3, List.cons(Nat.0, List.nil[Nat]))) = Nat.3
    sum(List.cons(Nat.2, List.cons(Nat.3, List.cons(Nat.0, List.nil[Nat])))) =
        Nat.2 + Nat.3
    sum(List.cons(Nat.0, List.cons(Nat.2, List.cons(Nat.3, List.cons(Nat.0, List.nil[Nat]))))) =
        Nat.0 + (Nat.2 + Nat.3)
    neg_sum_value
    Nat.0 + (Nat.2 + Nat.3) = Nat.5
    sum(map(divisor_list(Nat.6), jordan_summand_neg(Nat.1, Nat.6))) = Nat.5
    jordan_neg(Nat.1, Nat.6) = Nat.5
    // J_1(6) = 7 - 5 = 2.
    jordan_totient(Nat.1, Nat.6) = Nat.7 - Nat.5
    seven_minus_five
    Nat.7 - Nat.5 = Nat.2
    jordan_totient(Nat.1, Nat.6) = Nat.2
}

/// Euler's totient at six is two: `phi(6) = 2`.
theorem totient_six {
    Nat.6.totient = Nat.2
} by {
    two_is_prime
    three_is_prime
    three_ne_two
    Nat.3 != Nat.2
    Nat.2 != Nat.3
    totient_pq(Nat.2, Nat.3)
    (Nat.2 * Nat.3).totient = (Nat.2 - Nat.1) * (Nat.3 - Nat.1)
    Nat.2 * Nat.3 = Nat.6
    Nat.6.totient = (Nat.2 - Nat.1) * (Nat.3 - Nat.1)
    Nat.2 - Nat.1 = Nat.1
    Nat.3 - Nat.1 = Nat.2
    (Nat.2 - Nat.1) * (Nat.3 - Nat.1) = Nat.1 * Nat.2
    Nat.1 * Nat.2 = Nat.2
    (Nat.2 - Nat.1) * (Nat.3 - Nat.1) = Nat.2
    Nat.6.totient = Nat.2
}

/// The Jordan totient at `k = 1` agrees with Euler's totient at six:
/// `J_1(6) = phi(6) = 2`.
theorem jordan_totient_one_eq_totient_six {
    jordan_totient(Nat.1, Nat.6) = Nat.6.totient
} by {
    jordan_totient_six
    jordan_totient(Nat.1, Nat.6) = Nat.2
    totient_six
    Nat.6.totient = Nat.2
    jordan_totient(Nat.1, Nat.6) = Nat.6.totient
}

// /// The Jordan totient at `k = 1` is Euler's totient: `J_1(n) = phi(n)`.
// /// This is the classical Möbius-inversion identity
// /// `sum_{d | n} mu(d) * (n / d) = phi(n)`; the general proof is left for
// /// future work, the case `n = 6` being verified above.
// theorem jordan_totient_one_eq_totient(n: Nat) {
//     jordan_totient(Nat.1, n) = n.totient
// }

// /// Dedekind psi is multiplicative: `psi(m * n) = psi(m) * psi(n)` whenever
// /// `m` and `n` are coprime.
// ///
// /// From the product form `psi(n) = n * prod_{p | n} (1 + 1/p)` the prime
// /// divisors of a coprime product are disjoint, so the product splits into
// /// `psi(m) * psi(n)`.  A direct proof from the squarefree-divisor sum is
// /// left for future work.
// theorem dedekind_psi_multiplicative(m: Nat, n: Nat) {
//     m.coprime(n) implies dedekind_psi(m * n) = dedekind_psi(m) * dedekind_psi(n)
// }
