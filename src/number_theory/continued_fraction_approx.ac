from nat import Nat, zero_or_suc, lt_or_lte, mul_two_left, lte_cancel_suc, add_assoc, add_comm, add_one_right, add_suc_right, distrib_left, distrib_right, from_nat_one, from_nat_zero, lt_imp_lte_suc, lt_mul_both, lt_suc, lte_add_left, lte_add_right, lte_mul_both, lte_trans, mul_assoc, mul_comm, mul_suc_right, mul_zero_left, mul_zero_right, add_to_zero, pos_of_ne_zero, mul_one_left, mul_one_right, lt_not_ref, add_zero_right, add_zero_left
from int import Int, abs, add_from_nat, mul_from_nat, abs_neg, abs_from_nat
from rat import Rat, cross_mul_lt, cross_mul_lte, from_nat_add, from_nat_mul, nat_lt_imp_rat_lt, sub_div_distrib, add_div_distrib, mul_fractions, cancel_left_num_denom, neg_num, neg_denom, add_inv_cancels_left, add_inv_cancels_right, sub_self, rat_total, pos_imp_zero_lt, zero_lt_imp_pos, pos_inverse, pos_ne_zero, recip_recip, recip_mul, mul_div_cancels, div_zero, add_pos_pos, mul_pos_pos, nat_lte_imp_rat_lte, lte_cancel_mul_pos, abs_div, iop, iop_pos, iop_gets_lt, lt_some_nat, neg_recip, div_neg_neg, div_negs_cancel, zero_recip, neg_abs, abs_nonneg, recip_eq_one_div, zero_lte_abs, neg_pos_is_neg, neg_neg_is_pos, lt_add_pos, sub_from_int
from pair import Pair, pair_new_first, pair_new_second
from algebra.ring.ring import alternating_sign, alternating_sign_zero, alternating_sign_suc
from number_theory.continued_fraction_convergents import continued_fraction_recurrence_state, continued_fraction_recurrence_state_zero, continued_fraction_recurrence_state_suc, continued_fraction_recurrence_state_suc_first, continued_fraction_recurrence_state_suc_second, continued_fraction_convergent_numerator, continued_fraction_convergent_denominator, continued_fraction_convergent_value, positive_continued_fraction_sequence_tail, continued_fraction_convergent_numerator_zero, continued_fraction_convergent_denominator_zero, continued_fraction_convergent_numerator_suc, continued_fraction_convergent_denominator_suc, continued_fraction_convergent_denominator_positive, continued_fraction_adjacent_convergent_determinant_identity, continued_fraction_adjacent_cross_product_of_positive_sign, continued_fraction_adjacent_cross_product_of_negative_sign, continued_fraction_alternating_sign_double, continued_fraction_alternating_sign_double_suc, continued_fraction_convergent_value_lt_next_of_positive_sign, continued_fraction_convergent_value_gt_next_of_negative_sign, continued_fraction_even_convergent_lt_following_odd, continued_fraction_following_even_lt_odd_convergent, continued_fraction_nat_ratio_lt_of_cross_product, continued_fraction_convergent_value_lt_next_of_cross_product, continued_fraction_next_value_lt_of_cross_product
from number_theory.continued_fraction import nat_mul_positive, nat_add_positive_left, nat_positive_ne_zero
from real import Real, add_from_rat, from_nat_is_from_rat, real_from_rat_inverse, from_rat_maintains_lt, from_rat_maintains_lte, mul_from_rat, neg_from_rat, converges, limit, converges_to, converges_to_imp_converges, converges_imp_converges_to, convergent_converges_to_limit, converges_to_unique, tail_bound, tail_bound_implies_is_close, close_imp_bounds, close_comm, eps_lt_half, add_seq, seq_lte, seq_lte_preserves_limit, const_converges, const_converges_to, const_limit, limit_add_seq, is_upper_bound, is_increasing, increasing_convergent_bounded_by_limit, monotone_convergence_principle, abs_from_rat, pos_imp_eq_abs, abs_gte_zero, mul_abs, triangle_ineq, lt_some_int, gt_some_int, neg_is_close, rat_between_reals, gt_zero_imp_pos, pos_gt_zero, lt_imp_minus_pos, lt_trans, lt_add_right, lt_add_left, lte_abs, close_and_lt_imp_close
from ordered_field import inverse_on_positive_flips_inequality
from order import lt_of_lt_of_lte, lt_of_lte_of_lt, lt_imp_lte, lte_imp_not_lt

numerals Nat

/// The alternating sign is one at every index.
/// The alternating sign at an index is either one or negative one.
theorem continued_fraction_alternating_sign_one_or_neg_one(n: Nat) {
    alternating_sign[Int](n) = Int.1 or alternating_sign[Int](n) = -Int.1
} by {
    define p(k: Nat) -> Bool {
        alternating_sign[Int](k) = Int.1 or alternating_sign[Int](k) = -Int.1
    }
    alternating_sign_zero[Int]
    p(Nat.0)
    forall(k: Nat) {
        if p(k) {
            alternating_sign_suc[Int](k)
            if alternating_sign[Int](k) = Int.1 {
                alternating_sign[Int](k.suc) = -Int.1
                p(k.suc)
            }
            if alternating_sign[Int](k) = -Int.1 {
                alternating_sign[Int](k.suc) = -(-Int.1)
                --Int.1 = Int.1
                alternating_sign[Int](k.suc) = Int.1
                p(k.suc)
            }
            alternating_sign[Int](k) = Int.1 or alternating_sign[Int](k) = -Int.1
            p(k.suc)
        }
        p(k) implies p(k.suc)
    }
    p(Nat.0) and forall(k: Nat) { p(k) implies p(k.suc) }
    Nat.induction(p)
    forall(k: Nat) { p(k) }
    p(n)
}

/// The absolute value of the alternating sign is one.
theorem continued_fraction_alternating_sign_abs_one(n: Nat) {
    abs(alternating_sign[Int](n)) = Nat.1
} by {
    continued_fraction_alternating_sign_one_or_neg_one(n)
    from_nat_one[Int]
    if alternating_sign[Int](n) = Int.1 {
        abs(alternating_sign[Int](n)) = abs(Int.1)
        Int.from_nat(Nat.1) = Int.1
        abs_from_nat(Nat.1)
        abs(Int.from_nat(Nat.1)) = Nat.1
        abs(Int.1) = Nat.1
        abs(alternating_sign[Int](n)) = Nat.1
    }
    if alternating_sign[Int](n) = -Int.1 {
        abs_neg(Int.1)
        abs(-Int.1) = abs(Int.1)
        from_nat_one[Int]
        abs_from_nat(Nat.1)
        abs(Int.from_nat(Nat.1)) = Nat.1
        abs(Int.1) = Nat.1
        abs(-Int.1) = Nat.1
        abs(alternating_sign[Int](n)) = Nat.1
    }
}

/// The determinant of adjacent convergents in the classical form
/// `p_{n-1} q_n - p_n q_{n-1}` is the negation of the alternating sign.
theorem continued_fraction_convergent_determinant_standard(
    coefficients: Nat -> Nat, n: Nat
) {
    Int.from_nat(continued_fraction_convergent_numerator(coefficients, n)) *
            Int.from_nat(continued_fraction_convergent_denominator(
                coefficients, n.suc)) -
        Int.from_nat(continued_fraction_convergent_numerator(
            coefficients, n.suc)) *
            Int.from_nat(continued_fraction_convergent_denominator(
                coefficients, n)) =
        -alternating_sign[Int](n)
} by {
    continued_fraction_adjacent_convergent_determinant_identity(coefficients, n)
    Int.from_nat(continued_fraction_convergent_numerator(coefficients, n.suc)) *
            Int.from_nat(continued_fraction_convergent_denominator(
                coefficients, n)) -
        Int.from_nat(continued_fraction_convergent_numerator(coefficients, n)) *
            Int.from_nat(continued_fraction_convergent_denominator(
                coefficients, n.suc)) =
        alternating_sign[Int](n)
    -(Int.from_nat(continued_fraction_convergent_numerator(coefficients, n.suc)) *
            Int.from_nat(continued_fraction_convergent_denominator(
                coefficients, n)) -
        Int.from_nat(continued_fraction_convergent_numerator(coefficients, n)) *
            Int.from_nat(continued_fraction_convergent_denominator(
                coefficients, n.suc))) =
        -(alternating_sign[Int](n))
    Int.from_nat(continued_fraction_convergent_numerator(coefficients, n)) *
            Int.from_nat(continued_fraction_convergent_denominator(
                coefficients, n.suc)) -
        Int.from_nat(continued_fraction_convergent_numerator(
            coefficients, n.suc)) *
            Int.from_nat(continued_fraction_convergent_denominator(
                coefficients, n)) =
        -(Int.from_nat(continued_fraction_convergent_numerator(coefficients, n.suc)) *
                Int.from_nat(continued_fraction_convergent_denominator(
                    coefficients, n)) -
            Int.from_nat(continued_fraction_convergent_numerator(coefficients, n)) *
                Int.from_nat(continued_fraction_convergent_denominator(
                    coefficients, n.suc)))
    Int.from_nat(continued_fraction_convergent_numerator(coefficients, n)) *
            Int.from_nat(continued_fraction_convergent_denominator(
                coefficients, n.suc)) -
        Int.from_nat(continued_fraction_convergent_numerator(
            coefficients, n.suc)) *
            Int.from_nat(continued_fraction_convergent_denominator(
                coefficients, n)) =
        -alternating_sign[Int](n)
}

/// The absolute value of the classical adjacent-convergent determinant is one.
theorem continued_fraction_convergent_determinant_abs_one(
    coefficients: Nat -> Nat, n: Nat
) {
    abs(Int.from_nat(continued_fraction_convergent_numerator(coefficients, n)) *
            Int.from_nat(continued_fraction_convergent_denominator(
                coefficients, n.suc)) -
        Int.from_nat(continued_fraction_convergent_numerator(
            coefficients, n.suc)) *
            Int.from_nat(continued_fraction_convergent_denominator(
                coefficients, n))) =
        Nat.1
} by {
    continued_fraction_convergent_determinant_standard(coefficients, n)
    continued_fraction_alternating_sign_abs_one(n)
    abs(-alternating_sign[Int](n)) = abs(alternating_sign[Int](n))
    abs(Int.from_nat(continued_fraction_convergent_numerator(coefficients, n)) *
            Int.from_nat(continued_fraction_convergent_denominator(
                coefficients, n.suc)) -
        Int.from_nat(continued_fraction_convergent_numerator(
            coefficients, n.suc)) *
            Int.from_nat(continued_fraction_convergent_denominator(
                coefficients, n))) =
        abs(-alternating_sign[Int](n))
    abs(alternating_sign[Int](n)) = Nat.1
    abs(Int.from_nat(continued_fraction_convergent_numerator(coefficients, n)) *
            Int.from_nat(continued_fraction_convergent_denominator(
                coefficients, n.suc)) -
        Int.from_nat(continued_fraction_convergent_numerator(
            coefficients, n.suc)) *
            Int.from_nat(continued_fraction_convergent_denominator(
                coefficients, n))) =
        Nat.1
}

/// The numerator two steps ahead satisfies the continued-fraction recurrence.
theorem continued_fraction_convergent_numerator_two_suc(
    coefficients: Nat -> Nat, n: Nat
) {
    continued_fraction_convergent_numerator(coefficients, n.suc.suc) =
        continued_fraction_convergent_numerator(coefficients, n.suc) *
            coefficients(n.suc.suc) +
        continued_fraction_convergent_numerator(coefficients, n)
} by {
    continued_fraction_recurrence_state_suc_second(
        coefficients, n.suc.suc, Nat.0, Nat.1)
    continued_fraction_recurrence_state(
        coefficients, n.suc.suc.suc, Nat.0, Nat.1).second =
        continued_fraction_recurrence_state(
            coefficients, n.suc.suc, Nat.0, Nat.1).second *
            coefficients(n.suc.suc) +
        continued_fraction_recurrence_state(
            coefficients, n.suc.suc, Nat.0, Nat.1).first
    continued_fraction_convergent_numerator(coefficients, n.suc.suc) =
        continued_fraction_recurrence_state(
            coefficients, n.suc.suc.suc, Nat.0, Nat.1).second
    continued_fraction_convergent_numerator(coefficients, n.suc.suc) =
        continued_fraction_recurrence_state(
            coefficients, n.suc.suc, Nat.0, Nat.1).second *
            coefficients(n.suc.suc) +
        continued_fraction_recurrence_state(
            coefficients, n.suc.suc, Nat.0, Nat.1).first
    continued_fraction_convergent_numerator(coefficients, n.suc) =
        continued_fraction_recurrence_state(
            coefficients, n.suc.suc, Nat.0, Nat.1).second
    continued_fraction_recurrence_state_suc_first(
        coefficients, n.suc, Nat.0, Nat.1)
    continued_fraction_recurrence_state(
        coefficients, n.suc.suc, Nat.0, Nat.1).first =
        continued_fraction_recurrence_state(
            coefficients, n.suc, Nat.0, Nat.1).second
    continued_fraction_convergent_numerator(coefficients, n) =
        continued_fraction_recurrence_state(
            coefficients, n.suc, Nat.0, Nat.1).second
    continued_fraction_convergent_numerator(coefficients, n.suc.suc) =
        continued_fraction_convergent_numerator(coefficients, n.suc) *
            coefficients(n.suc.suc) +
        continued_fraction_convergent_numerator(coefficients, n)
}

/// The denominator two steps ahead satisfies the continued-fraction recurrence.
theorem continued_fraction_convergent_denominator_two_suc(
    coefficients: Nat -> Nat, n: Nat
) {
    continued_fraction_convergent_denominator(coefficients, n.suc.suc) =
        continued_fraction_convergent_denominator(coefficients, n.suc) *
            coefficients(n.suc.suc) +
        continued_fraction_convergent_denominator(coefficients, n)
} by {
    continued_fraction_recurrence_state_suc_second(
        coefficients, n.suc.suc, Nat.1, Nat.0)
    continued_fraction_recurrence_state(
        coefficients, n.suc.suc.suc, Nat.1, Nat.0).second =
        continued_fraction_recurrence_state(
            coefficients, n.suc.suc, Nat.1, Nat.0).second *
            coefficients(n.suc.suc) +
        continued_fraction_recurrence_state(
            coefficients, n.suc.suc, Nat.1, Nat.0).first
    continued_fraction_convergent_denominator(coefficients, n.suc.suc) =
        continued_fraction_recurrence_state(
            coefficients, n.suc.suc.suc, Nat.1, Nat.0).second
    continued_fraction_convergent_denominator(coefficients, n.suc.suc) =
        continued_fraction_recurrence_state(
            coefficients, n.suc.suc, Nat.1, Nat.0).second *
            coefficients(n.suc.suc) +
        continued_fraction_recurrence_state(
            coefficients, n.suc.suc, Nat.1, Nat.0).first
    continued_fraction_convergent_denominator(coefficients, n.suc) =
        continued_fraction_recurrence_state(
            coefficients, n.suc.suc, Nat.1, Nat.0).second
    continued_fraction_recurrence_state_suc_first(
        coefficients, n.suc, Nat.1, Nat.0)
    continued_fraction_recurrence_state(
        coefficients, n.suc.suc, Nat.1, Nat.0).first =
        continued_fraction_recurrence_state(
            coefficients, n.suc, Nat.1, Nat.0).second
    continued_fraction_convergent_denominator(coefficients, n) =
        continued_fraction_recurrence_state(
            coefficients, n.suc, Nat.1, Nat.0).second
    continued_fraction_convergent_denominator(coefficients, n.suc.suc) =
        continued_fraction_convergent_denominator(coefficients, n.suc) *
            coefficients(n.suc.suc) +
        continued_fraction_convergent_denominator(coefficients, n)
}

/// The difference of two rational fractions with nonzero denominators.
theorem rat_sub_fraction(numerator1: Rat, denominator1: Rat, numerator2: Rat,
    denominator2: Rat) {
    denominator1 != Rat.0 and denominator2 != Rat.0 implies (numerator1 / denominator1) - (numerator2 / denominator2) = (numerator1 * denominator2 - numerator2 * denominator1) / (denominator1 * denominator2)
} by {
    if denominator1 != Rat.0 and denominator2 != Rat.0 {
        cancel_left_num_denom(denominator2, numerator1, denominator1)
        (denominator2 * numerator1) / (denominator2 * denominator1) =
            numerator1 / denominator1
        numerator1 / denominator1 =
            (numerator1 * denominator2) / (denominator1 * denominator2)
        cancel_left_num_denom(denominator1, -numerator2, denominator2)
        (denominator1 * -numerator2) / (denominator1 * denominator2) =
            -numerator2 / denominator2
        (-numerator2) / denominator2 =
            ((-numerator2) * denominator1) / (denominator2 * denominator1)
        (numerator1 / denominator1) - (numerator2 / denominator2) =
            (numerator1 / denominator1) + (-(numerator2 / denominator2))
        -(numerator2 / denominator2) = (-numerator2) / denominator2
        (numerator1 / denominator1) + (-(numerator2 / denominator2)) =
            (numerator1 / denominator1) + (-numerator2) / denominator2
        (numerator1 / denominator1) + (-numerator2) / denominator2 =
            (numerator1 * denominator2) / (denominator1 * denominator2) +
                ((-numerator2) * denominator1) / (denominator2 * denominator1)
        add_div_distrib(numerator1 * denominator2, (-numerator2) * denominator1,
            denominator1 * denominator2)
        ((numerator1 * denominator2) + (-numerator2) * denominator1) /
                (denominator1 * denominator2) =
            (numerator1 * denominator2) / (denominator1 * denominator2) +
                ((-numerator2) * denominator1) / (denominator1 * denominator2)
        (numerator1 * denominator2) / (denominator1 * denominator2) +
                ((-numerator2) * denominator1) / (denominator2 * denominator1) =
            (numerator1 * denominator2) / (denominator1 * denominator2) +
                ((-numerator2) * denominator1) / (denominator1 * denominator2)
        (numerator1 / denominator1) + (-numerator2) / denominator2 =
            ((numerator1 * denominator2) + (-numerator2) * denominator1) /
                (denominator1 * denominator2)
        ((numerator1 * denominator2) + (-numerator2) * denominator1) /
                (denominator1 * denominator2) =
            (numerator1 * denominator2 - numerator2 * denominator1) /
                (denominator1 * denominator2)
        (numerator1 / denominator1) - (numerator2 / denominator2) =
            (numerator1 * denominator2 - numerator2 * denominator1) /
                (denominator1 * denominator2)
    }
}

/// A positive natural number embeds as a nonzero rational.
theorem rat_from_nat_positive_ne_zero(n: Nat) {
    Nat.0 < n implies Rat.from_nat(n) != Rat.0
} by {
    if Nat.0 < n {
        nat_lt_imp_rat_lt(Nat.0, n)
        Rat.from_nat(Nat.0) < Rat.from_nat(n)
        from_nat_zero[Rat]
        Rat.from_nat(Nat.0) = Rat.0
        Rat.0 < Rat.from_nat(n)
        zero_lt_imp_pos(Rat.from_nat(n))
        Rat.from_nat(n).is_positive
        pos_ne_zero(Rat.from_nat(n))
        Rat.from_nat(n) != Rat.0
    }
}

/// The signed difference of adjacent convergent values as a single fraction.
theorem continued_fraction_convergent_value_suc_sub(
    coefficients: Nat -> Nat, n: Nat
) {
    positive_continued_fraction_sequence_tail(coefficients) implies continued_fraction_convergent_value(coefficients, n.suc) - continued_fraction_convergent_value(coefficients, n) = (Rat.from_nat(continued_fraction_convergent_numerator(
                    coefficients, n.suc)) *
                Rat.from_nat(continued_fraction_convergent_denominator(
                    coefficients, n)) -
                Rat.from_nat(continued_fraction_convergent_numerator(
                    coefficients, n)) *
                Rat.from_nat(continued_fraction_convergent_denominator(
                    coefficients, n.suc))) /
            (Rat.from_nat(continued_fraction_convergent_denominator(
                    coefficients, n)) *
                Rat.from_nat(continued_fraction_convergent_denominator(
                    coefficients, n.suc)))
} by {
    if positive_continued_fraction_sequence_tail(coefficients) {
        continued_fraction_convergent_denominator_positive(coefficients, n)
        Nat.0 < continued_fraction_convergent_denominator(coefficients, n)
        rat_from_nat_positive_ne_zero(
            continued_fraction_convergent_denominator(coefficients, n))
        Rat.from_nat(continued_fraction_convergent_denominator(coefficients, n)) != Rat.0
        continued_fraction_convergent_denominator_positive(coefficients, n.suc)
        rat_from_nat_positive_ne_zero(
            continued_fraction_convergent_denominator(coefficients, n.suc))
        Rat.from_nat(continued_fraction_convergent_denominator(
            coefficients, n.suc)) != Rat.0
        rat_sub_fraction(
            Rat.from_nat(continued_fraction_convergent_numerator(
                coefficients, n.suc)),
            Rat.from_nat(continued_fraction_convergent_denominator(
                coefficients, n.suc)),
            Rat.from_nat(continued_fraction_convergent_numerator(
                coefficients, n)),
            Rat.from_nat(continued_fraction_convergent_denominator(
                coefficients, n)))
        continued_fraction_convergent_value(coefficients, n.suc) =
            Rat.from_nat(continued_fraction_convergent_numerator(
                coefficients, n.suc)) /
            Rat.from_nat(continued_fraction_convergent_denominator(
                coefficients, n.suc))
        continued_fraction_convergent_value(coefficients, n) =
            Rat.from_nat(continued_fraction_convergent_numerator(
                coefficients, n)) /
            Rat.from_nat(continued_fraction_convergent_denominator(
                coefficients, n))
        continued_fraction_convergent_value(coefficients, n.suc) -
                continued_fraction_convergent_value(coefficients, n) =
            (Rat.from_nat(continued_fraction_convergent_numerator(
                    coefficients, n.suc)) *
                Rat.from_nat(continued_fraction_convergent_denominator(
                    coefficients, n)) -
                Rat.from_nat(continued_fraction_convergent_numerator(
                    coefficients, n)) *
                Rat.from_nat(continued_fraction_convergent_denominator(
                    coefficients, n.suc))) /
            (Rat.from_nat(continued_fraction_convergent_denominator(
                    coefficients, n)) *
                Rat.from_nat(continued_fraction_convergent_denominator(
                    coefficients, n.suc)))
    }
}

/// At a positive alternating sign, the difference of the adjacent numerator
/// products is one.
theorem continued_fraction_numerator_difference_of_positive_sign(
    coefficients: Nat -> Nat, n: Nat
) {
    alternating_sign[Int](n) = Int.1 implies Rat.from_nat(continued_fraction_convergent_numerator(coefficients, n.suc)) * Rat.from_nat(continued_fraction_convergent_denominator(
                    coefficients, n)) -
            Rat.from_nat(continued_fraction_convergent_numerator(coefficients, n)) *
                Rat.from_nat(continued_fraction_convergent_denominator(
                    coefficients, n.suc)) =
            Rat.1
} by {
    if alternating_sign[Int](n) = Int.1 {
        continued_fraction_adjacent_cross_product_of_positive_sign(coefficients, n)
        continued_fraction_convergent_numerator(coefficients, n) *
                continued_fraction_convergent_denominator(coefficients, n.suc) +
                Nat.1 =
            continued_fraction_convergent_numerator(coefficients, n.suc) *
                continued_fraction_convergent_denominator(coefficients, n)
        from_nat_add(
            continued_fraction_convergent_numerator(coefficients, n) *
                continued_fraction_convergent_denominator(coefficients, n.suc),
            Nat.1)
        Rat.from_nat(continued_fraction_convergent_numerator(coefficients, n) *
                continued_fraction_convergent_denominator(coefficients, n.suc) +
                Nat.1) =
            Rat.from_nat(continued_fraction_convergent_numerator(coefficients, n) *
                continued_fraction_convergent_denominator(coefficients, n.suc)) +
            Rat.from_nat(Nat.1)
        from_nat_one[Rat]
        Rat.from_nat(Nat.1) = Rat.1
        Rat.from_nat(continued_fraction_convergent_numerator(coefficients, n.suc) *
                continued_fraction_convergent_denominator(coefficients, n)) =
            Rat.from_nat(continued_fraction_convergent_numerator(coefficients, n) *
                continued_fraction_convergent_denominator(coefficients, n.suc)) +
            Rat.1
        from_nat_mul(continued_fraction_convergent_numerator(coefficients, n.suc),
            continued_fraction_convergent_denominator(coefficients, n))
        from_nat_mul(continued_fraction_convergent_numerator(coefficients, n),
            continued_fraction_convergent_denominator(coefficients, n.suc))
        Rat.from_nat(continued_fraction_convergent_numerator(coefficients, n.suc)) *
                Rat.from_nat(continued_fraction_convergent_denominator(
                    coefficients, n)) =
            Rat.from_nat(continued_fraction_convergent_numerator(coefficients, n)) *
                    Rat.from_nat(continued_fraction_convergent_denominator(
                        coefficients, n.suc)) +
                Rat.1
        Rat.from_nat(continued_fraction_convergent_numerator(coefficients, n.suc)) *
                Rat.from_nat(continued_fraction_convergent_denominator(
                    coefficients, n)) -
            Rat.from_nat(continued_fraction_convergent_numerator(coefficients, n)) *
                Rat.from_nat(continued_fraction_convergent_denominator(
                    coefficients, n.suc)) =
            (Rat.from_nat(continued_fraction_convergent_numerator(coefficients, n)) *
                    Rat.from_nat(continued_fraction_convergent_denominator(
                        coefficients, n.suc)) +
                Rat.1) -
            Rat.from_nat(continued_fraction_convergent_numerator(coefficients, n)) *
                Rat.from_nat(continued_fraction_convergent_denominator(
                    coefficients, n.suc))
        (Rat.from_nat(continued_fraction_convergent_numerator(coefficients, n)) *
                Rat.from_nat(continued_fraction_convergent_denominator(
                    coefficients, n.suc)) +
            Rat.1) -
            Rat.from_nat(continued_fraction_convergent_numerator(coefficients, n)) *
                Rat.from_nat(continued_fraction_convergent_denominator(
                    coefficients, n.suc)) =
            Rat.1
        Rat.from_nat(continued_fraction_convergent_numerator(coefficients, n.suc)) *
                Rat.from_nat(continued_fraction_convergent_denominator(
                    coefficients, n)) -
            Rat.from_nat(continued_fraction_convergent_numerator(coefficients, n)) *
                Rat.from_nat(continued_fraction_convergent_denominator(
                    coefficients, n.suc)) =
            Rat.1
    }
}

/// At a negative alternating sign, the difference of the adjacent numerator
/// products is negative one.
theorem continued_fraction_numerator_difference_of_negative_sign(
    coefficients: Nat -> Nat, n: Nat
) {
    alternating_sign[Int](n) = -Int.1 implies Rat.from_nat(continued_fraction_convergent_numerator(coefficients, n.suc)) * Rat.from_nat(continued_fraction_convergent_denominator(
                    coefficients, n)) -
            Rat.from_nat(continued_fraction_convergent_numerator(coefficients, n)) *
                Rat.from_nat(continued_fraction_convergent_denominator(
                    coefficients, n.suc)) =
            -Rat.1
} by {
    if alternating_sign[Int](n) = -Int.1 {
        continued_fraction_adjacent_cross_product_of_negative_sign(coefficients, n)
        continued_fraction_convergent_numerator(coefficients, n.suc) *
                continued_fraction_convergent_denominator(coefficients, n) +
                Nat.1 =
            continued_fraction_convergent_numerator(coefficients, n) *
                continued_fraction_convergent_denominator(coefficients, n.suc)
        from_nat_add(
            continued_fraction_convergent_numerator(coefficients, n.suc) *
                continued_fraction_convergent_denominator(coefficients, n),
            Nat.1)
        Rat.from_nat(continued_fraction_convergent_numerator(coefficients, n.suc) *
                continued_fraction_convergent_denominator(coefficients, n) +
                Nat.1) =
            Rat.from_nat(continued_fraction_convergent_numerator(coefficients, n.suc) *
                continued_fraction_convergent_denominator(coefficients, n)) +
            Rat.from_nat(Nat.1)
        from_nat_one[Rat]
        Rat.from_nat(Nat.1) = Rat.1
        Rat.from_nat(continued_fraction_convergent_numerator(coefficients, n) *
                continued_fraction_convergent_denominator(coefficients, n.suc)) =
            Rat.from_nat(continued_fraction_convergent_numerator(coefficients, n.suc) *
                continued_fraction_convergent_denominator(coefficients, n)) +
            Rat.1
        from_nat_mul(continued_fraction_convergent_numerator(coefficients, n.suc),
            continued_fraction_convergent_denominator(coefficients, n))
        from_nat_mul(continued_fraction_convergent_numerator(coefficients, n),
            continued_fraction_convergent_denominator(coefficients, n.suc))
        Rat.from_nat(continued_fraction_convergent_numerator(coefficients, n)) *
                Rat.from_nat(continued_fraction_convergent_denominator(
                    coefficients, n.suc)) =
            Rat.from_nat(continued_fraction_convergent_numerator(coefficients, n.suc)) *
                    Rat.from_nat(continued_fraction_convergent_denominator(
                        coefficients, n)) +
                Rat.1
        Rat.from_nat(continued_fraction_convergent_numerator(coefficients, n.suc)) *
                Rat.from_nat(continued_fraction_convergent_denominator(
                    coefficients, n)) -
            Rat.from_nat(continued_fraction_convergent_numerator(coefficients, n)) *
                Rat.from_nat(continued_fraction_convergent_denominator(
                    coefficients, n.suc)) =
            Rat.from_nat(continued_fraction_convergent_numerator(coefficients, n.suc)) *
                    Rat.from_nat(continued_fraction_convergent_denominator(
                        coefficients, n)) -
                (Rat.from_nat(continued_fraction_convergent_numerator(
                        coefficients, n.suc)) *
                    Rat.from_nat(continued_fraction_convergent_denominator(
                        coefficients, n)) +
                    Rat.1)
        Rat.from_nat(continued_fraction_convergent_numerator(coefficients, n.suc)) *
                Rat.from_nat(continued_fraction_convergent_denominator(
                    coefficients, n)) -
            (Rat.from_nat(continued_fraction_convergent_numerator(coefficients, n.suc)) *
                    Rat.from_nat(continued_fraction_convergent_denominator(
                        coefficients, n)) +
                Rat.1) =
            -Rat.1
        Rat.from_nat(continued_fraction_convergent_numerator(coefficients, n.suc)) *
                Rat.from_nat(continued_fraction_convergent_denominator(
                    coefficients, n)) -
            Rat.from_nat(continued_fraction_convergent_numerator(coefficients, n)) *
                Rat.from_nat(continued_fraction_convergent_denominator(
                    coefficients, n.suc)) =
            -Rat.1
    }
}

/// Negation distributes over rational division.
theorem rat_neg_div(a: Rat, b: Rat) {
    (-a) / b = -(a / b)
} by {
}

/// The reciprocal of the product of the denominators of two adjacent
/// convergents.
define continued_fraction_gap_bound(coefficients: Nat -> Nat, n: Nat) -> Rat {
    Rat.1 / Rat.from_nat(
        continued_fraction_convergent_denominator(coefficients, n) *
        continued_fraction_convergent_denominator(coefficients, n.suc))
}

/// The reciprocal of the product of adjacent denominators is positive.
theorem continued_fraction_gap_bound_positive(coefficients: Nat -> Nat, n: Nat) {
    positive_continued_fraction_sequence_tail(coefficients) implies continued_fraction_gap_bound(coefficients, n).is_positive
} by {
    if positive_continued_fraction_sequence_tail(coefficients) {
        continued_fraction_convergent_denominator_positive(coefficients, n)
        Nat.0 < continued_fraction_convergent_denominator(coefficients, n)
        continued_fraction_convergent_denominator_positive(coefficients, n.suc)
        Nat.0 < continued_fraction_convergent_denominator(coefficients, n.suc)
        nat_mul_positive(
            continued_fraction_convergent_denominator(coefficients, n),
            continued_fraction_convergent_denominator(coefficients, n.suc))
        Nat.0 < continued_fraction_convergent_denominator(coefficients, n) and
            Nat.0 < continued_fraction_convergent_denominator(coefficients, n.suc)
        Nat.0 < continued_fraction_convergent_denominator(coefficients, n) *
            continued_fraction_convergent_denominator(coefficients, n.suc)
        rat_from_nat_positive_ne_zero(
            continued_fraction_convergent_denominator(coefficients, n) *
            continued_fraction_convergent_denominator(coefficients, n.suc))
        Rat.from_nat(continued_fraction_convergent_denominator(coefficients, n) *
            continued_fraction_convergent_denominator(coefficients, n.suc)) != Rat.0
        recip_eq_one_div(Rat.from_nat(
            continued_fraction_convergent_denominator(coefficients, n) *
            continued_fraction_convergent_denominator(coefficients, n.suc)))
        Rat.from_nat(continued_fraction_convergent_denominator(coefficients, n) *
            continued_fraction_convergent_denominator(coefficients, n.suc)).inverse =
            Rat.1 / Rat.from_nat(continued_fraction_convergent_denominator(
                coefficients, n) *
                continued_fraction_convergent_denominator(coefficients, n.suc))
        nat_lt_imp_rat_lt(Nat.0,
            continued_fraction_convergent_denominator(coefficients, n) *
            continued_fraction_convergent_denominator(coefficients, n.suc))
        Rat.from_nat(Nat.0) < Rat.from_nat(
            continued_fraction_convergent_denominator(coefficients, n) *
            continued_fraction_convergent_denominator(coefficients, n.suc))
        from_nat_zero[Rat]
        Rat.from_nat(Nat.0) = Rat.0
        Rat.0 < Rat.from_nat(continued_fraction_convergent_denominator(
            coefficients, n) *
            continued_fraction_convergent_denominator(coefficients, n.suc))
        zero_lt_imp_pos(Rat.from_nat(continued_fraction_convergent_denominator(
            coefficients, n) *
            continued_fraction_convergent_denominator(coefficients, n.suc)))
        Rat.from_nat(continued_fraction_convergent_denominator(coefficients, n) *
            continued_fraction_convergent_denominator(coefficients, n.suc)).is_positive
        pos_inverse(Rat.from_nat(continued_fraction_convergent_denominator(
            coefficients, n) *
            continued_fraction_convergent_denominator(coefficients, n.suc)))
        (Rat.from_nat(continued_fraction_convergent_denominator(coefficients, n) *
            continued_fraction_convergent_denominator(coefficients, n.suc))).inverse.is_positive
        (Rat.1 / Rat.from_nat(continued_fraction_convergent_denominator(
            coefficients, n) *
            continued_fraction_convergent_denominator(coefficients, n.suc))) .is_positive
        continued_fraction_gap_bound(coefficients, n).is_positive
    }
}

/// The absolute difference of adjacent convergent values.
define continued_fraction_convergent_gap(coefficients: Nat -> Nat, n: Nat) -> Rat {
    (continued_fraction_convergent_value(coefficients, n.suc) -
        continued_fraction_convergent_value(coefficients, n)).abs
}

/// At a positive alternating sign, the absolute difference of adjacent
/// convergent values is the reciprocal of the product of their denominators.
theorem continued_fraction_convergent_gap_eq_reciprocal_product_of_positive_sign(
    coefficients: Nat -> Nat, n: Nat
) {
    positive_continued_fraction_sequence_tail(coefficients) and
        alternating_sign[Int](n) = Int.1 implies continued_fraction_convergent_gap(coefficients, n) = continued_fraction_gap_bound(coefficients, n)
} by {
    if positive_continued_fraction_sequence_tail(coefficients) and
            alternating_sign[Int](n) = Int.1 {
        continued_fraction_convergent_value_suc_sub(coefficients, n)
        continued_fraction_gap_bound_positive(coefficients, n)
        continued_fraction_numerator_difference_of_positive_sign(coefficients, n)
        from_nat_mul(
            continued_fraction_convergent_denominator(coefficients, n),
            continued_fraction_convergent_denominator(coefficients, n.suc))
        Rat.from_nat(continued_fraction_convergent_denominator(coefficients, n)) *
                Rat.from_nat(continued_fraction_convergent_denominator(
                    coefficients, n.suc)) =
            Rat.from_nat(continued_fraction_convergent_denominator(coefficients, n) *
                continued_fraction_convergent_denominator(coefficients, n.suc))
        continued_fraction_convergent_value(coefficients, n.suc) -
                continued_fraction_convergent_value(coefficients, n) =
            Rat.1 / Rat.from_nat(
                continued_fraction_convergent_denominator(coefficients, n) *
                continued_fraction_convergent_denominator(coefficients, n.suc))
        continued_fraction_convergent_value(coefficients, n.suc) -
                continued_fraction_convergent_value(coefficients, n) =
            continued_fraction_gap_bound(coefficients, n)
        continued_fraction_gap_bound(coefficients, n).is_positive
        not (continued_fraction_convergent_value(coefficients, n.suc) -
            continued_fraction_convergent_value(coefficients, n)).is_negative
        (continued_fraction_convergent_value(coefficients, n.suc) -
            continued_fraction_convergent_value(coefficients, n)).abs =
            continued_fraction_convergent_value(coefficients, n.suc) -
            continued_fraction_convergent_value(coefficients, n)
        continued_fraction_convergent_gap(coefficients, n) =
            continued_fraction_gap_bound(coefficients, n)
    }
}

/// At a negative alternating sign, the absolute difference of adjacent
/// convergent values is the reciprocal of the product of their denominators.
theorem continued_fraction_convergent_gap_eq_reciprocal_product_of_negative_sign(
    coefficients: Nat -> Nat, n: Nat
) {
    positive_continued_fraction_sequence_tail(coefficients) and
        alternating_sign[Int](n) = -Int.1 implies continued_fraction_convergent_gap(coefficients, n) = continued_fraction_gap_bound(coefficients, n)
} by {
    if positive_continued_fraction_sequence_tail(coefficients) and
            alternating_sign[Int](n) = -Int.1 {
        continued_fraction_convergent_value_suc_sub(coefficients, n)
        continued_fraction_gap_bound_positive(coefficients, n)
        continued_fraction_numerator_difference_of_negative_sign(coefficients, n)
        from_nat_mul(
            continued_fraction_convergent_denominator(coefficients, n),
            continued_fraction_convergent_denominator(coefficients, n.suc))
        Rat.from_nat(continued_fraction_convergent_denominator(coefficients, n)) *
                Rat.from_nat(continued_fraction_convergent_denominator(
                    coefficients, n.suc)) =
            Rat.from_nat(continued_fraction_convergent_denominator(coefficients, n) *
                continued_fraction_convergent_denominator(coefficients, n.suc))
        continued_fraction_convergent_value(coefficients, n.suc) -
                continued_fraction_convergent_value(coefficients, n) =
            -Rat.1 / Rat.from_nat(
                continued_fraction_convergent_denominator(coefficients, n) *
                continued_fraction_convergent_denominator(coefficients, n.suc))
        rat_neg_div(Rat.1, Rat.from_nat(
            continued_fraction_convergent_denominator(coefficients, n) *
            continued_fraction_convergent_denominator(coefficients, n.suc)))
        -Rat.1 / Rat.from_nat(
            continued_fraction_convergent_denominator(coefficients, n) *
            continued_fraction_convergent_denominator(coefficients, n.suc)) =
            -(Rat.1 / Rat.from_nat(
                continued_fraction_convergent_denominator(coefficients, n) *
                continued_fraction_convergent_denominator(coefficients, n.suc)))
        continued_fraction_convergent_value(coefficients, n.suc) -
                continued_fraction_convergent_value(coefficients, n) =
            -(Rat.1 / Rat.from_nat(
                continued_fraction_convergent_denominator(coefficients, n) *
                continued_fraction_convergent_denominator(coefficients, n.suc)))
        continued_fraction_convergent_value(coefficients, n.suc) -
                continued_fraction_convergent_value(coefficients, n) =
            -continued_fraction_gap_bound(coefficients, n)
        neg_pos_is_neg(continued_fraction_gap_bound(coefficients, n))
        (-continued_fraction_gap_bound(coefficients, n)).is_negative
        (continued_fraction_convergent_value(coefficients, n.suc) -
            continued_fraction_convergent_value(coefficients, n)).is_negative
        (continued_fraction_convergent_value(coefficients, n.suc) -
            continued_fraction_convergent_value(coefficients, n)).abs =
            -(continued_fraction_convergent_value(coefficients, n.suc) -
                continued_fraction_convergent_value(coefficients, n))
        -(continued_fraction_convergent_value(coefficients, n.suc) -
            continued_fraction_convergent_value(coefficients, n)) =
            continued_fraction_gap_bound(coefficients, n)
        continued_fraction_convergent_gap(coefficients, n) =
            continued_fraction_gap_bound(coefficients, n)
    }
}

/// The absolute difference of adjacent convergent values is the reciprocal of
/// the product of their denominators.
theorem continued_fraction_convergent_gap_eq_reciprocal_product(
    coefficients: Nat -> Nat, n: Nat
) {
    positive_continued_fraction_sequence_tail(coefficients) implies continued_fraction_convergent_gap(coefficients, n) = continued_fraction_gap_bound(coefficients, n)
} by {
    if positive_continued_fraction_sequence_tail(coefficients) {
        continued_fraction_alternating_sign_one_or_neg_one(n)
        alternating_sign[Int](n) = Int.1 or alternating_sign[Int](n) = -Int.1
        if alternating_sign[Int](n) = Int.1 {
            continued_fraction_convergent_gap_eq_reciprocal_product_of_positive_sign(
                coefficients, n)
            continued_fraction_convergent_gap(coefficients, n) =
                continued_fraction_gap_bound(coefficients, n)
        }
        if alternating_sign[Int](n) = -Int.1 {
            continued_fraction_convergent_gap_eq_reciprocal_product_of_negative_sign(
                coefficients, n)
            continued_fraction_convergent_gap(coefficients, n) =
                continued_fraction_gap_bound(coefficients, n)
        }
    }
}

/// Every convergent denominator is at least one.
theorem continued_fraction_convergent_denominator_ge_one(
    coefficients: Nat -> Nat, n: Nat
) {
    positive_continued_fraction_sequence_tail(coefficients) implies Nat.1 <= continued_fraction_convergent_denominator(coefficients, n)
} by {
    if positive_continued_fraction_sequence_tail(coefficients) {
        continued_fraction_convergent_denominator_positive(coefficients, n)
        Nat.0 < continued_fraction_convergent_denominator(coefficients, n)
        lt_imp_lte_suc(Nat.0, continued_fraction_convergent_denominator(coefficients, n))
        Nat.1 <= continued_fraction_convergent_denominator(coefficients, n)
    }
}

/// The first convergent denominator is at least one.
theorem continued_fraction_convergent_denominator_one_ge(
    coefficients: Nat -> Nat
) {
    positive_continued_fraction_sequence_tail(coefficients) implies Nat.1 <= continued_fraction_convergent_denominator(coefficients, Nat.1)
} by {
    if positive_continued_fraction_sequence_tail(coefficients) {
        continued_fraction_convergent_denominator_suc(coefficients, Nat.0)
        continued_fraction_convergent_denominator(coefficients, Nat.1) =
            continued_fraction_convergent_denominator(coefficients, Nat.0) *
                coefficients(Nat.1) +
            continued_fraction_recurrence_state(
                coefficients, Nat.1, Nat.1, Nat.0).first
        continued_fraction_convergent_denominator_zero(coefficients)
        continued_fraction_recurrence_state_suc_first(
            coefficients, Nat.0, Nat.1, Nat.0)
        continued_fraction_recurrence_state(
            coefficients, Nat.1, Nat.1, Nat.0).first =
            continued_fraction_recurrence_state(
                coefficients, Nat.0, Nat.1, Nat.0).second
        continued_fraction_recurrence_state_zero(
            coefficients, Nat.1, Nat.0)
        continued_fraction_recurrence_state(
            coefficients, Nat.0, Nat.1, Nat.0) = Pair.new(Nat.1, Nat.0)
        continued_fraction_recurrence_state(
            coefficients, Nat.0, Nat.1, Nat.0).second = Nat.0
        continued_fraction_recurrence_state(
            coefficients, Nat.1, Nat.1, Nat.0).first = Nat.0
        continued_fraction_convergent_denominator(coefficients, Nat.1) =
            Nat.1 * coefficients(Nat.1) + Nat.0
        mul_one_left(coefficients(Nat.1))
        Nat.1 * coefficients(Nat.1) = coefficients(Nat.1)
        continued_fraction_convergent_denominator(coefficients, Nat.1) =
            coefficients(Nat.1)
        Nat.0 < coefficients(Nat.1)
        lt_imp_lte_suc(Nat.0, coefficients(Nat.1))
        Nat.1 <= coefficients(Nat.1)
        Nat.1 <= continued_fraction_convergent_denominator(coefficients, Nat.1)
    }
}

/// The denominator two steps ahead exceeds the one-step-ahead denominator by
/// at least one.
theorem continued_fraction_convergent_denominator_suc_ge_one_more(
    coefficients: Nat -> Nat, m: Nat
) {
    positive_continued_fraction_sequence_tail(coefficients) implies continued_fraction_convergent_denominator(coefficients, m.suc) + Nat.1 <= continued_fraction_convergent_denominator(coefficients, m.suc.suc)
} by {
    if positive_continued_fraction_sequence_tail(coefficients) {
        continued_fraction_convergent_denominator_suc(coefficients, m.suc)
        continued_fraction_convergent_denominator(coefficients, m.suc.suc) =
            continued_fraction_convergent_denominator(coefficients, m.suc) *
                coefficients(m.suc.suc) +
            continued_fraction_recurrence_state(
                coefficients, m.suc.suc, Nat.1, Nat.0).first
        continued_fraction_recurrence_state_suc_first(
            coefficients, m.suc, Nat.1, Nat.0)
        continued_fraction_recurrence_state(
            coefficients, m.suc.suc, Nat.1, Nat.0).first =
            continued_fraction_recurrence_state(
                coefficients, m.suc, Nat.1, Nat.0).second
        continued_fraction_convergent_denominator(coefficients, m) =
            continued_fraction_recurrence_state(
                coefficients, m.suc, Nat.1, Nat.0).second
        continued_fraction_convergent_denominator(coefficients, m.suc.suc) =
            continued_fraction_convergent_denominator(coefficients, m.suc) *
                coefficients(m.suc.suc) +
            continued_fraction_convergent_denominator(coefficients, m)
        Nat.0 < coefficients(m.suc.suc)
        lt_imp_lte_suc(Nat.0, coefficients(m.suc.suc))
        Nat.1 <= coefficients(m.suc.suc)
        continued_fraction_convergent_denominator_ge_one(coefficients, m.suc)
        Nat.1 <= continued_fraction_convergent_denominator(coefficients, m.suc)
        lte_mul_both(
            continued_fraction_convergent_denominator(coefficients, m.suc),
            Nat.1, coefficients(m.suc.suc))
        Nat.1 * continued_fraction_convergent_denominator(coefficients, m.suc) <= coefficients(m.suc.suc) * continued_fraction_convergent_denominator(coefficients, m.suc)
        mul_one_left(continued_fraction_convergent_denominator(coefficients, m.suc))
        Nat.1 * continued_fraction_convergent_denominator(coefficients, m.suc) =
            continued_fraction_convergent_denominator(coefficients, m.suc)
        continued_fraction_convergent_denominator(coefficients, m.suc) <= coefficients(m.suc.suc) * continued_fraction_convergent_denominator(coefficients, m.suc)
        continued_fraction_convergent_denominator(coefficients, m.suc) <= continued_fraction_convergent_denominator(coefficients, m.suc) * coefficients(m.suc.suc)
        continued_fraction_convergent_denominator_ge_one(coefficients, m)
        Nat.1 <= continued_fraction_convergent_denominator(coefficients, m)
        lte_add_right(
            Nat.1, continued_fraction_convergent_denominator(coefficients, m),
            continued_fraction_convergent_denominator(coefficients, m.suc))
        Nat.1 + continued_fraction_convergent_denominator(coefficients, m.suc) <= continued_fraction_convergent_denominator(coefficients, m) + continued_fraction_convergent_denominator(coefficients, m.suc)
        continued_fraction_convergent_denominator(coefficients, m.suc) + Nat.1 <= continued_fraction_convergent_denominator(coefficients, m.suc) + continued_fraction_convergent_denominator(coefficients, m)
        lte_add_right(
            continued_fraction_convergent_denominator(coefficients, m.suc),
            continued_fraction_convergent_denominator(coefficients, m.suc) *
                coefficients(m.suc.suc),
            continued_fraction_convergent_denominator(coefficients, m))
        continued_fraction_convergent_denominator(coefficients, m.suc) +
                continued_fraction_convergent_denominator(coefficients, m) <= continued_fraction_convergent_denominator(coefficients, m.suc) * coefficients(m.suc.suc) + continued_fraction_convergent_denominator(coefficients, m)
        lte_trans(
            continued_fraction_convergent_denominator(coefficients, m.suc) + Nat.1,
            continued_fraction_convergent_denominator(coefficients, m.suc) +
                continued_fraction_convergent_denominator(coefficients, m),
            continued_fraction_convergent_denominator(coefficients, m.suc.suc))
        continued_fraction_convergent_denominator(coefficients, m.suc) + Nat.1 <= continued_fraction_convergent_denominator(coefficients, m.suc.suc)
    }
}

/// Adjacent convergent denominators strictly increase.
theorem continued_fraction_convergent_denominator_suc_gt(
    coefficients: Nat -> Nat, m: Nat
) {
    positive_continued_fraction_sequence_tail(coefficients) implies continued_fraction_convergent_denominator(coefficients, m.suc) < continued_fraction_convergent_denominator(coefficients, m.suc.suc)
} by {
    if positive_continued_fraction_sequence_tail(coefficients) {
        continued_fraction_convergent_denominator_suc_ge_one_more(coefficients, m)
        continued_fraction_convergent_denominator(coefficients, m.suc) + Nat.1 <= continued_fraction_convergent_denominator(coefficients, m.suc.suc)
        add_one_right(continued_fraction_convergent_denominator(coefficients, m.suc))
        continued_fraction_convergent_denominator(coefficients, m.suc) + Nat.1 =
            continued_fraction_convergent_denominator(coefficients, m.suc).suc
        lt_suc(continued_fraction_convergent_denominator(coefficients, m.suc))
        continued_fraction_convergent_denominator(coefficients, m.suc) < continued_fraction_convergent_denominator(coefficients, m.suc).suc
        continued_fraction_convergent_denominator(coefficients, m.suc) < continued_fraction_convergent_denominator(coefficients, m.suc) + Nat.1
        lt_of_lt_of_lte(
            continued_fraction_convergent_denominator(coefficients, m.suc),
            continued_fraction_convergent_denominator(coefficients, m.suc) + Nat.1,
            continued_fraction_convergent_denominator(coefficients, m.suc.suc))
        continued_fraction_convergent_denominator(coefficients, m.suc) < continued_fraction_convergent_denominator(coefficients, m.suc.suc)
    }
}

/// The index is a lower bound for the convergent denominator.
theorem continued_fraction_convergent_denominator_ge_index(
    coefficients: Nat -> Nat, n: Nat
) {
    positive_continued_fraction_sequence_tail(coefficients) implies n <= continued_fraction_convergent_denominator(coefficients, n)
} by {
    if positive_continued_fraction_sequence_tail(coefficients) {
        define p(m: Nat) -> Bool {
            m.suc <= continued_fraction_convergent_denominator(coefficients, m.suc)
        }
        continued_fraction_convergent_denominator_one_ge(coefficients)
        Nat.1 <= continued_fraction_convergent_denominator(coefficients, Nat.1)
        p(Nat.0)
        forall(m: Nat) {
            if p(m) {
                continued_fraction_convergent_denominator_suc_ge_one_more(
                    coefficients, m)
                continued_fraction_convergent_denominator(coefficients, m.suc) + Nat.1 <= continued_fraction_convergent_denominator(
                        coefficients, m.suc.suc)
                m.suc <= continued_fraction_convergent_denominator(
                    coefficients, m.suc)
                lte_add_right(
                    m.suc, continued_fraction_convergent_denominator(
                        coefficients, m.suc), Nat.1)
                m.suc + Nat.1 <= continued_fraction_convergent_denominator(
                        coefficients, m.suc) + Nat.1
                lte_trans(
                    m.suc + Nat.1,
                    continued_fraction_convergent_denominator(
                        coefficients, m.suc) + Nat.1,
                    continued_fraction_convergent_denominator(
                        coefficients, m.suc.suc))
                m.suc + Nat.1 <= continued_fraction_convergent_denominator(
                    coefficients, m.suc.suc)
                add_one_right(m.suc)
                m.suc + Nat.1 = m.suc.suc
                m.suc.suc <= continued_fraction_convergent_denominator(
                    coefficients, m.suc.suc)
                p(m.suc)
            }
            p(m) implies p(m.suc)
        }
        p(Nat.0) and forall(m: Nat) { p(m) implies p(m.suc) }
        Nat.induction(p)
        forall(m: Nat) { p(m) }
        zero_or_suc(n)
        n = Nat.0 or exists(m: Nat) { m.suc = n }
        if n = Nat.0 {
            continued_fraction_convergent_denominator_zero(coefficients)
            continued_fraction_convergent_denominator(coefficients, Nat.0) = Nat.1
            Nat.0 <= continued_fraction_convergent_denominator(coefficients, Nat.0)
            n <= continued_fraction_convergent_denominator(coefficients, n)
            n = Nat.0 implies n <= continued_fraction_convergent_denominator(coefficients, n)
        }
        if exists(m: Nat) { m.suc = n } {
            let m: Nat satisfy {
                m.suc = n
            }
            p(m)
            m.suc <= continued_fraction_convergent_denominator(coefficients, m.suc)
            n <= continued_fraction_convergent_denominator(coefficients, n)
            (exists(k: Nat) { k.suc = n }) implies n <= continued_fraction_convergent_denominator(coefficients, n)
        }
        n <= continued_fraction_convergent_denominator(coefficients, n)
    }
}

/// The real value of the convergent at an index.
define continued_fraction_real_convergent_value(
    coefficients: Nat -> Nat, n: Nat
) -> Real {
    Real.from_rat(continued_fraction_convergent_value(coefficients, n))
}

/// The real reciprocal of the product of two adjacent denominators.
define continued_fraction_real_gap_bound(coefficients: Nat -> Nat, n: Nat) -> Real {
    Real.from_rat(continued_fraction_gap_bound(coefficients, n))
}

/// The rational-to-real embedding respects subtraction.
theorem real_from_rat_sub(p: Rat, q: Rat) {
    Real.from_rat(p - q) = Real.from_rat(p) - Real.from_rat(q)
} by {
    neg_from_rat(q)
    Real.from_rat(-q) = -Real.from_rat(q)
    add_from_rat(p, -q)
    Real.from_rat(p + -q) = Real.from_rat(p) + Real.from_rat(-q)
    Real.from_rat(p) + Real.from_rat(-q) = Real.from_rat(p) - Real.from_rat(q)
    p - q = p + -q
    Real.from_rat(p - q) = Real.from_rat(p) - Real.from_rat(q)
}

/// The real distance between adjacent convergents is the embedded rational
/// gap bound.
theorem continued_fraction_real_gap_eq_embedded_gap(
    coefficients: Nat -> Nat, n: Nat
) {
    positive_continued_fraction_sequence_tail(coefficients) implies (continued_fraction_real_convergent_value(coefficients, n.suc) - continued_fraction_real_convergent_value(coefficients, n)).abs = continued_fraction_real_gap_bound(coefficients, n)
} by {
    if positive_continued_fraction_sequence_tail(coefficients) {
        continued_fraction_convergent_gap_eq_reciprocal_product(coefficients, n)
        continued_fraction_convergent_gap(coefficients, n) =
            continued_fraction_gap_bound(coefficients, n)
        abs_from_rat(continued_fraction_convergent_value(coefficients, n.suc) -
            continued_fraction_convergent_value(coefficients, n))
        Real.from_rat(continued_fraction_convergent_gap(coefficients, n)).abs =
            Real.from_rat((continued_fraction_convergent_value(coefficients, n.suc) -
                continued_fraction_convergent_value(coefficients, n)).abs)
        real_from_rat_sub(continued_fraction_convergent_value(coefficients, n.suc),
            continued_fraction_convergent_value(coefficients, n))
        Real.from_rat(continued_fraction_convergent_value(coefficients, n.suc) -
            continued_fraction_convergent_value(coefficients, n)) =
            Real.from_rat(continued_fraction_convergent_value(coefficients, n.suc)) -
            Real.from_rat(continued_fraction_convergent_value(coefficients, n))
        (Real.from_rat(continued_fraction_convergent_value(coefficients, n.suc) -
            continued_fraction_convergent_value(coefficients, n))).abs =
            (Real.from_rat(continued_fraction_convergent_value(coefficients, n.suc)) -
                Real.from_rat(continued_fraction_convergent_value(coefficients, n))).abs
        (continued_fraction_real_convergent_value(coefficients, n.suc) -
            continued_fraction_real_convergent_value(coefficients, n)).abs =
            Real.from_rat(continued_fraction_convergent_gap(coefficients, n))
        (continued_fraction_real_convergent_value(coefficients, n.suc) -
            continued_fraction_real_convergent_value(coefficients, n)).abs =
            Real.from_rat(continued_fraction_gap_bound(coefficients, n))
        (continued_fraction_real_convergent_value(coefficients, n.suc) -
            continued_fraction_real_convergent_value(coefficients, n)).abs =
            continued_fraction_real_gap_bound(coefficients, n)
    }
}

/// The real gap bound is positive.
theorem continued_fraction_real_gap_bound_positive(
    coefficients: Nat -> Nat, n: Nat
) {
    positive_continued_fraction_sequence_tail(coefficients) implies continued_fraction_real_gap_bound(coefficients, n).is_positive
} by {
    if positive_continued_fraction_sequence_tail(coefficients) {
        continued_fraction_gap_bound_positive(coefficients, n)
        continued_fraction_gap_bound(coefficients, n).is_positive
        pos_imp_zero_lt(continued_fraction_gap_bound(coefficients, n))
        Rat.0 < continued_fraction_gap_bound(coefficients, n)
        from_rat_maintains_lt(Rat.0, continued_fraction_gap_bound(coefficients, n))
        Real.from_rat(Rat.0) < Real.from_rat(continued_fraction_gap_bound(coefficients, n))
        Real.from_rat(Rat.0) = Real.0
        Real.0 < Real.from_rat(continued_fraction_gap_bound(coefficients, n))
        gt_zero_imp_pos(Real.from_rat(continued_fraction_gap_bound(coefficients, n)))
        Real.from_rat(continued_fraction_gap_bound(coefficients, n)).is_positive
        continued_fraction_real_gap_bound(coefficients, n).is_positive
    }
}

/// A common summand cancels in the difference of embedded naturals.
theorem rat_from_nat_sub_add_cancel(n1: Nat, n2: Nat, n3: Nat) {
    Rat.from_nat(n1 + n3) - Rat.from_nat(n2 + n3) =
        Rat.from_nat(n1) - Rat.from_nat(n2)
} by {
    add_from_nat(n1, n3)
    add_from_nat(n2, n3)
    Int.from_nat(n1 + n3) = Int.from_nat(n1) + Int.from_nat(n3)
    Int.from_nat(n2 + n3) = Int.from_nat(n2) + Int.from_nat(n3)
    Int.from_nat(n1) + Int.from_nat(n3) - (Int.from_nat(n2) + Int.from_nat(n3)) =
        Int.from_nat(n1) - Int.from_nat(n2)
    sub_from_int(
        Int.from_nat(n1) + Int.from_nat(n3),
        Int.from_nat(n2) + Int.from_nat(n3))
    Rat.from_int(Int.from_nat(n1) + Int.from_nat(n3)) -
        Rat.from_int(Int.from_nat(n2) + Int.from_nat(n3)) =
        Rat.from_int(Int.from_nat(n1) + Int.from_nat(n3) -
            (Int.from_nat(n2) + Int.from_nat(n3)))
    sub_from_int(Int.from_nat(n1), Int.from_nat(n2))
    Rat.from_int(Int.from_nat(n1)) - Rat.from_int(Int.from_nat(n2)) =
        Rat.from_int(Int.from_nat(n1) - Int.from_nat(n2))
    Rat.from_nat(n1 + n3) = Rat.from_int(Int.from_nat(n1 + n3))
    Rat.from_nat(n2 + n3) = Rat.from_int(Int.from_nat(n2 + n3))
    Rat.from_nat(n1) = Rat.from_int(Int.from_nat(n1))
    Rat.from_nat(n2) = Rat.from_int(Int.from_nat(n2))
    Rat.from_nat(n1 + n3) - Rat.from_nat(n2 + n3) =
        Rat.from_nat(n1) - Rat.from_nat(n2)
}

/// The embedding of a threefold natural product is the product of the
/// embeddings.
theorem rat_from_nat_mul3(a: Nat, b: Nat, c: Nat) {
    Rat.from_nat(a * b * c) = Rat.from_nat(a) * Rat.from_nat(b) * Rat.from_nat(c)
} by {
    from_nat_mul(a * b, c)
    from_nat_mul(a, b)
    Rat.from_nat(a * b * c) = Rat.from_nat(a * b) * Rat.from_nat(c)
    Rat.from_nat(a * b) = Rat.from_nat(a) * Rat.from_nat(b)
    Rat.from_nat(a * b * c) = Rat.from_nat(a) * Rat.from_nat(b) * Rat.from_nat(c)
}

/// A common factor distributes out of a rational difference.
theorem rat_sub_factor(a: Rat, x: Rat, y: Rat) {
    a * x - a * y = a * (x - y)
} by {
}

/// The cross product of the numerators and denominators two steps apart is
/// the next coefficient times the adjacent cross product.
theorem continued_fraction_convergent_cross_two_suc(
    coefficients: Nat -> Nat, n: Nat
) {
    Rat.from_nat(continued_fraction_convergent_numerator(coefficients, n.suc.suc) *
            continued_fraction_convergent_denominator(coefficients, n)) -
        Rat.from_nat(continued_fraction_convergent_numerator(coefficients, n) *
            continued_fraction_convergent_denominator(coefficients, n.suc.suc)) =
        Rat.from_nat(coefficients(n.suc.suc)) *
            (Rat.from_nat(continued_fraction_convergent_numerator(
                    coefficients, n.suc) *
                continued_fraction_convergent_denominator(coefficients, n)) -
                Rat.from_nat(continued_fraction_convergent_numerator(
                    coefficients, n) *
                continued_fraction_convergent_denominator(
                    coefficients, n.suc)))
} by {
    continued_fraction_convergent_numerator_two_suc(coefficients, n)
    continued_fraction_convergent_denominator_two_suc(coefficients, n)
    continued_fraction_convergent_numerator(coefficients, n.suc.suc) =
        continued_fraction_convergent_numerator(coefficients, n.suc) *
            coefficients(n.suc.suc) +
        continued_fraction_convergent_numerator(coefficients, n)
    continued_fraction_convergent_denominator(coefficients, n.suc.suc) =
        continued_fraction_convergent_denominator(coefficients, n.suc) *
            coefficients(n.suc.suc) +
        continued_fraction_convergent_denominator(coefficients, n)
    distrib_right(
        continued_fraction_convergent_numerator(coefficients, n.suc) *
            coefficients(n.suc.suc),
        continued_fraction_convergent_numerator(coefficients, n),
        continued_fraction_convergent_denominator(coefficients, n))
    (continued_fraction_convergent_numerator(coefficients, n.suc) *
            coefficients(n.suc.suc) +
        continued_fraction_convergent_numerator(coefficients, n)) *
        continued_fraction_convergent_denominator(coefficients, n) =
        continued_fraction_convergent_numerator(coefficients, n.suc) *
                coefficients(n.suc.suc) *
                continued_fraction_convergent_denominator(coefficients, n) +
            continued_fraction_convergent_numerator(coefficients, n) *
                continued_fraction_convergent_denominator(coefficients, n)
    distrib_left(
        continued_fraction_convergent_numerator(coefficients, n),
        continued_fraction_convergent_denominator(coefficients, n.suc) *
            coefficients(n.suc.suc),
        continued_fraction_convergent_denominator(coefficients, n))
    continued_fraction_convergent_numerator(coefficients, n) *
            (continued_fraction_convergent_denominator(coefficients, n.suc) *
                coefficients(n.suc.suc) +
            continued_fraction_convergent_denominator(coefficients, n)) =
        continued_fraction_convergent_numerator(coefficients, n) *
                (continued_fraction_convergent_denominator(coefficients, n.suc) *
                    coefficients(n.suc.suc)) +
            continued_fraction_convergent_numerator(coefficients, n) *
                continued_fraction_convergent_denominator(coefficients, n)
    continued_fraction_convergent_numerator(coefficients, n) *
            continued_fraction_convergent_denominator(coefficients, n.suc.suc) =
        continued_fraction_convergent_numerator(coefficients, n) *
            (continued_fraction_convergent_denominator(coefficients, n.suc) *
                coefficients(n.suc.suc) +
            continued_fraction_convergent_denominator(coefficients, n))
    continued_fraction_convergent_numerator(coefficients, n) *
            continued_fraction_convergent_denominator(coefficients, n.suc.suc) =
        continued_fraction_convergent_numerator(coefficients, n) *
                (continued_fraction_convergent_denominator(coefficients, n.suc) *
                    coefficients(n.suc.suc)) +
            continued_fraction_convergent_numerator(coefficients, n) *
                continued_fraction_convergent_denominator(coefficients, n)
    rat_from_nat_sub_add_cancel(
        continued_fraction_convergent_numerator(coefficients, n.suc) *
            coefficients(n.suc.suc) *
            continued_fraction_convergent_denominator(coefficients, n),
        continued_fraction_convergent_numerator(coefficients, n) *
            (continued_fraction_convergent_denominator(coefficients, n.suc) *
                coefficients(n.suc.suc)),
        continued_fraction_convergent_numerator(coefficients, n) *
            continued_fraction_convergent_denominator(coefficients, n))
    Rat.from_nat(continued_fraction_convergent_numerator(coefficients, n.suc) *
            coefficients(n.suc.suc) *
            continued_fraction_convergent_denominator(coefficients, n) +
        continued_fraction_convergent_numerator(coefficients, n) *
            continued_fraction_convergent_denominator(coefficients, n)) -
        Rat.from_nat(continued_fraction_convergent_numerator(coefficients, n) *
                (continued_fraction_convergent_denominator(coefficients, n.suc) *
                    coefficients(n.suc.suc)) +
            continued_fraction_convergent_numerator(coefficients, n) *
                continued_fraction_convergent_denominator(coefficients, n)) =
        Rat.from_nat(continued_fraction_convergent_numerator(coefficients, n.suc) *
                coefficients(n.suc.suc) *
                continued_fraction_convergent_denominator(coefficients, n)) -
        Rat.from_nat(continued_fraction_convergent_numerator(coefficients, n) *
            (continued_fraction_convergent_denominator(coefficients, n.suc) *
                coefficients(n.suc.suc)))
    mul_assoc(
        continued_fraction_convergent_numerator(coefficients, n.suc),
        coefficients(n.suc.suc),
        continued_fraction_convergent_denominator(coefficients, n))
    continued_fraction_convergent_numerator(coefficients, n.suc) *
            coefficients(n.suc.suc) *
            continued_fraction_convergent_denominator(coefficients, n) =
        continued_fraction_convergent_numerator(coefficients, n.suc) *
            (coefficients(n.suc.suc) *
                continued_fraction_convergent_denominator(coefficients, n))
    mul_comm(coefficients(n.suc.suc),
        continued_fraction_convergent_denominator(coefficients, n))
    coefficients(n.suc.suc) *
            continued_fraction_convergent_denominator(coefficients, n) =
        continued_fraction_convergent_denominator(coefficients, n) *
            coefficients(n.suc.suc)
    mul_comm(continued_fraction_convergent_denominator(coefficients, n.suc),
        coefficients(n.suc.suc))
    continued_fraction_convergent_denominator(coefficients, n.suc) *
            coefficients(n.suc.suc) =
        coefficients(n.suc.suc) *
            continued_fraction_convergent_denominator(coefficients, n.suc)
    rat_from_nat_mul3(
        coefficients(n.suc.suc),
        continued_fraction_convergent_numerator(coefficients, n.suc),
        continued_fraction_convergent_denominator(coefficients, n))
    Rat.from_nat(coefficients(n.suc.suc) *
            continued_fraction_convergent_numerator(coefficients, n.suc) *
            continued_fraction_convergent_denominator(coefficients, n)) =
        Rat.from_nat(coefficients(n.suc.suc)) *
        Rat.from_nat(continued_fraction_convergent_numerator(coefficients, n.suc)) *
        Rat.from_nat(continued_fraction_convergent_denominator(coefficients, n))
    Rat.from_nat(continued_fraction_convergent_numerator(coefficients, n.suc) *
            coefficients(n.suc.suc) *
            continued_fraction_convergent_denominator(coefficients, n)) =
        Rat.from_nat(coefficients(n.suc.suc)) *
        Rat.from_nat(continued_fraction_convergent_numerator(coefficients, n.suc)) *
        Rat.from_nat(continued_fraction_convergent_denominator(coefficients, n))
    rat_from_nat_mul3(
        coefficients(n.suc.suc),
        continued_fraction_convergent_numerator(coefficients, n),
        continued_fraction_convergent_denominator(coefficients, n.suc))
    Rat.from_nat(coefficients(n.suc.suc) *
            continued_fraction_convergent_numerator(coefficients, n) *
            continued_fraction_convergent_denominator(coefficients, n.suc)) =
        Rat.from_nat(coefficients(n.suc.suc)) *
        Rat.from_nat(continued_fraction_convergent_numerator(coefficients, n)) *
        Rat.from_nat(continued_fraction_convergent_denominator(coefficients, n.suc))
    mul_comm(
        continued_fraction_convergent_denominator(coefficients, n.suc),
        coefficients(n.suc.suc))
    continued_fraction_convergent_denominator(coefficients, n.suc) *
            coefficients(n.suc.suc) =
        coefficients(n.suc.suc) *
        continued_fraction_convergent_denominator(coefficients, n.suc)
    continued_fraction_convergent_numerator(coefficients, n) *
            (continued_fraction_convergent_denominator(coefficients, n.suc) *
                coefficients(n.suc.suc)) =
        continued_fraction_convergent_numerator(coefficients, n) *
            (coefficients(n.suc.suc) *
                continued_fraction_convergent_denominator(coefficients, n.suc))
    mul_assoc(
        continued_fraction_convergent_numerator(coefficients, n),
        coefficients(n.suc.suc),
        continued_fraction_convergent_denominator(coefficients, n.suc))
    continued_fraction_convergent_numerator(coefficients, n) *
            (coefficients(n.suc.suc) *
                continued_fraction_convergent_denominator(coefficients, n.suc)) =
        continued_fraction_convergent_numerator(coefficients, n) *
            coefficients(n.suc.suc) *
            continued_fraction_convergent_denominator(coefficients, n.suc)
    mul_comm(
        continued_fraction_convergent_numerator(coefficients, n),
        coefficients(n.suc.suc))
    continued_fraction_convergent_numerator(coefficients, n) *
            coefficients(n.suc.suc) =
        coefficients(n.suc.suc) *
        continued_fraction_convergent_numerator(coefficients, n)
    continued_fraction_convergent_numerator(coefficients, n) *
            (continued_fraction_convergent_denominator(coefficients, n.suc) *
                coefficients(n.suc.suc)) =
        coefficients(n.suc.suc) *
        continued_fraction_convergent_numerator(coefficients, n) *
        continued_fraction_convergent_denominator(coefficients, n.suc)
    Rat.from_nat(continued_fraction_convergent_numerator(coefficients, n) *
            (continued_fraction_convergent_denominator(coefficients, n.suc) *
                coefficients(n.suc.suc))) =
        Rat.from_nat(coefficients(n.suc.suc)) *
        Rat.from_nat(continued_fraction_convergent_numerator(coefficients, n)) *
        Rat.from_nat(continued_fraction_convergent_denominator(coefficients, n.suc))
    Rat.from_nat(continued_fraction_convergent_numerator(coefficients, n.suc.suc) *
            continued_fraction_convergent_denominator(coefficients, n)) -
        Rat.from_nat(continued_fraction_convergent_numerator(coefficients, n) *
            continued_fraction_convergent_denominator(coefficients, n.suc.suc)) =
        Rat.from_nat(coefficients(n.suc.suc)) *
            (Rat.from_nat(continued_fraction_convergent_numerator(
                    coefficients, n.suc) *
                continued_fraction_convergent_denominator(coefficients, n)) -
                Rat.from_nat(continued_fraction_convergent_numerator(
                    coefficients, n) *
                continued_fraction_convergent_denominator(coefficients, n.suc)))
}

/// The reciprocal of the product of the denominators two steps apart,
/// scaled by the intermediate coefficient.
define continued_fraction_two_step_bound(coefficients: Nat -> Nat, n: Nat) -> Rat {
    Rat.from_nat(coefficients(n.suc.suc)) /
        (Rat.from_nat(continued_fraction_convergent_denominator(coefficients, n)) *
            Rat.from_nat(continued_fraction_convergent_denominator(
                coefficients, n.suc.suc)))
}

/// The signed difference of the convergent values two steps apart.
theorem continued_fraction_convergent_value_two_suc_sub(
    coefficients: Nat -> Nat, n: Nat
) {
    positive_continued_fraction_sequence_tail(coefficients) implies continued_fraction_convergent_value(coefficients, n.suc.suc) - continued_fraction_convergent_value(coefficients, n) = (Rat.from_nat(coefficients(n.suc.suc)) * (Rat.from_nat(continued_fraction_convergent_numerator(
                        coefficients, n.suc)) *
                    Rat.from_nat(continued_fraction_convergent_denominator(
                        coefficients, n)) -
                    Rat.from_nat(continued_fraction_convergent_numerator(
                        coefficients, n)) *
                    Rat.from_nat(continued_fraction_convergent_denominator(
                        coefficients, n.suc)))) /
            (Rat.from_nat(continued_fraction_convergent_denominator(
                    coefficients, n)) *
                Rat.from_nat(continued_fraction_convergent_denominator(
                    coefficients, n.suc.suc)))
} by {
    if positive_continued_fraction_sequence_tail(coefficients) {
        continued_fraction_convergent_denominator_positive(coefficients, n)
        continued_fraction_convergent_denominator_positive(coefficients, n.suc.suc)
        rat_from_nat_positive_ne_zero(
            continued_fraction_convergent_denominator(coefficients, n))
        Rat.from_nat(continued_fraction_convergent_denominator(coefficients, n)) != Rat.0
        rat_from_nat_positive_ne_zero(
            continued_fraction_convergent_denominator(coefficients, n.suc.suc))
        Rat.from_nat(continued_fraction_convergent_denominator(
            coefficients, n.suc.suc)) != Rat.0
        rat_sub_fraction(
            Rat.from_nat(continued_fraction_convergent_numerator(
                coefficients, n.suc.suc)),
            Rat.from_nat(continued_fraction_convergent_denominator(
                coefficients, n.suc.suc)),
            Rat.from_nat(continued_fraction_convergent_numerator(
                coefficients, n)),
            Rat.from_nat(continued_fraction_convergent_denominator(
                coefficients, n)))
        continued_fraction_convergent_value(coefficients, n.suc.suc) =
            Rat.from_nat(continued_fraction_convergent_numerator(
                coefficients, n.suc.suc)) /
            Rat.from_nat(continued_fraction_convergent_denominator(
                coefficients, n.suc.suc))
        continued_fraction_convergent_value(coefficients, n) =
            Rat.from_nat(continued_fraction_convergent_numerator(
                coefficients, n)) /
            Rat.from_nat(continued_fraction_convergent_denominator(
                coefficients, n))
        continued_fraction_convergent_value(coefficients, n.suc.suc) -
                continued_fraction_convergent_value(coefficients, n) =
            (Rat.from_nat(continued_fraction_convergent_numerator(
                    coefficients, n.suc.suc)) *
                Rat.from_nat(continued_fraction_convergent_denominator(
                    coefficients, n)) -
                Rat.from_nat(continued_fraction_convergent_numerator(
                    coefficients, n)) *
                Rat.from_nat(continued_fraction_convergent_denominator(
                    coefficients, n.suc.suc))) /
            (Rat.from_nat(continued_fraction_convergent_denominator(
                    coefficients, n)) *
                Rat.from_nat(continued_fraction_convergent_denominator(
                    coefficients, n.suc.suc)))
        continued_fraction_convergent_cross_two_suc(coefficients, n)
        Rat.from_nat(continued_fraction_convergent_numerator(coefficients, n.suc.suc) *
                continued_fraction_convergent_denominator(coefficients, n)) -
            Rat.from_nat(continued_fraction_convergent_numerator(coefficients, n) *
                continued_fraction_convergent_denominator(coefficients, n.suc.suc)) =
            Rat.from_nat(coefficients(n.suc.suc)) *
                (Rat.from_nat(continued_fraction_convergent_numerator(
                        coefficients, n.suc) *
                    continued_fraction_convergent_denominator(coefficients, n)) -
                    Rat.from_nat(continued_fraction_convergent_numerator(
                        coefficients, n) *
                    continued_fraction_convergent_denominator(
                        coefficients, n.suc)))
        from_nat_mul(
            continued_fraction_convergent_numerator(coefficients, n.suc.suc),
            continued_fraction_convergent_denominator(coefficients, n))
        from_nat_mul(
            continued_fraction_convergent_numerator(coefficients, n),
            continued_fraction_convergent_denominator(coefficients, n.suc.suc))
        from_nat_mul(
            continued_fraction_convergent_numerator(coefficients, n.suc),
            continued_fraction_convergent_denominator(coefficients, n))
        from_nat_mul(
            continued_fraction_convergent_numerator(coefficients, n),
            continued_fraction_convergent_denominator(coefficients, n.suc))
        Rat.from_nat(continued_fraction_convergent_numerator(coefficients, n.suc.suc)) *
                Rat.from_nat(continued_fraction_convergent_denominator(coefficients, n)) -
            Rat.from_nat(continued_fraction_convergent_numerator(coefficients, n)) *
                Rat.from_nat(continued_fraction_convergent_denominator(
                    coefficients, n.suc.suc)) =
            Rat.from_nat(coefficients(n.suc.suc)) *
                (Rat.from_nat(continued_fraction_convergent_numerator(
                        coefficients, n.suc)) *
                    Rat.from_nat(continued_fraction_convergent_denominator(
                        coefficients, n)) -
                    Rat.from_nat(continued_fraction_convergent_numerator(
                        coefficients, n)) *
                    Rat.from_nat(continued_fraction_convergent_denominator(
                        coefficients, n.suc)))
        continued_fraction_convergent_value(coefficients, n.suc.suc) -
                continued_fraction_convergent_value(coefficients, n) =
            (Rat.from_nat(coefficients(n.suc.suc)) *
                (Rat.from_nat(continued_fraction_convergent_numerator(
                        coefficients, n.suc)) *
                    Rat.from_nat(continued_fraction_convergent_denominator(
                        coefficients, n)) -
                    Rat.from_nat(continued_fraction_convergent_numerator(
                        coefficients, n)) *
                    Rat.from_nat(continued_fraction_convergent_denominator(
                        coefficients, n.suc)))) /
            (Rat.from_nat(continued_fraction_convergent_denominator(
                    coefficients, n)) *
                Rat.from_nat(continued_fraction_convergent_denominator(
                    coefficients, n.suc.suc)))
    }
}

/// The quotient of positive rationals is positive.
theorem rat_div_pos(a: Rat, b: Rat) {
    a.is_positive and b.is_positive implies (a / b).is_positive
} by {
    if a.is_positive and b.is_positive {
        pos_ne_zero(b)
        b != Rat.0
        recip_eq_one_div(b)
        b.inverse = Rat.1 / b
        pos_inverse(b)
        b.inverse.is_positive
        mul_pos_pos(a, b.inverse)
        a * b.inverse = a / b
        (a / b).is_positive
    }
}

/// At an even index, the convergent two steps ahead is strictly larger.
theorem continued_fraction_even_convergent_value_two_suc_gt(
    coefficients: Nat -> Nat, k: Nat
) {
    positive_continued_fraction_sequence_tail(coefficients) implies continued_fraction_convergent_value(coefficients, Nat.2 * k) < continued_fraction_convergent_value(coefficients, (Nat.2 * k).suc.suc)
} by {
    if positive_continued_fraction_sequence_tail(coefficients) {
        continued_fraction_alternating_sign_double(k)
        alternating_sign[Int](Nat.2 * k) = Int.1
        continued_fraction_numerator_difference_of_positive_sign(coefficients, Nat.2 * k)
        Rat.from_nat(continued_fraction_convergent_numerator(coefficients, (Nat.2 * k).suc)) *
                Rat.from_nat(continued_fraction_convergent_denominator(
                    coefficients, Nat.2 * k)) -
            Rat.from_nat(continued_fraction_convergent_numerator(
                coefficients, Nat.2 * k)) *
                Rat.from_nat(continued_fraction_convergent_denominator(
                    coefficients, (Nat.2 * k).suc)) =
            Rat.1
        continued_fraction_convergent_value_two_suc_sub(coefficients, Nat.2 * k)
        continued_fraction_convergent_value(coefficients, (Nat.2 * k).suc.suc) -
                continued_fraction_convergent_value(coefficients, Nat.2 * k) =
            (Rat.from_nat(coefficients((Nat.2 * k).suc.suc)) *
                (Rat.from_nat(continued_fraction_convergent_numerator(
                        coefficients, (Nat.2 * k).suc)) *
                    Rat.from_nat(continued_fraction_convergent_denominator(
                        coefficients, Nat.2 * k)) -
                    Rat.from_nat(continued_fraction_convergent_numerator(
                        coefficients, Nat.2 * k)) *
                    Rat.from_nat(continued_fraction_convergent_denominator(
                        coefficients, (Nat.2 * k).suc)))) /
            (Rat.from_nat(continued_fraction_convergent_denominator(
                    coefficients, Nat.2 * k)) *
                Rat.from_nat(continued_fraction_convergent_denominator(
                    coefficients, (Nat.2 * k).suc.suc)))
        continued_fraction_convergent_value(coefficients, (Nat.2 * k).suc.suc) -
                continued_fraction_convergent_value(coefficients, Nat.2 * k) =
            (Rat.from_nat(coefficients((Nat.2 * k).suc.suc)) * Rat.1) /
            (Rat.from_nat(continued_fraction_convergent_denominator(
                    coefficients, Nat.2 * k)) *
                Rat.from_nat(continued_fraction_convergent_denominator(
                    coefficients, (Nat.2 * k).suc.suc)))
        continued_fraction_convergent_value(coefficients, (Nat.2 * k).suc.suc) -
                continued_fraction_convergent_value(coefficients, Nat.2 * k) =
            Rat.from_nat(coefficients((Nat.2 * k).suc.suc)) /
            (Rat.from_nat(continued_fraction_convergent_denominator(
                    coefficients, Nat.2 * k)) *
                Rat.from_nat(continued_fraction_convergent_denominator(
                    coefficients, (Nat.2 * k).suc.suc)))
        Nat.0 < coefficients((Nat.2 * k).suc.suc)
        nat_lt_imp_rat_lt(Nat.0, coefficients((Nat.2 * k).suc.suc))
        Rat.from_nat(Nat.0) < Rat.from_nat(coefficients((Nat.2 * k).suc.suc))
        from_nat_zero[Rat]
        Rat.from_nat(Nat.0) = Rat.0
        Rat.0 < Rat.from_nat(coefficients((Nat.2 * k).suc.suc))
        zero_lt_imp_pos(Rat.from_nat(coefficients((Nat.2 * k).suc.suc)))
        Rat.from_nat(coefficients((Nat.2 * k).suc.suc)).is_positive
        continued_fraction_convergent_denominator_positive(coefficients, Nat.2 * k)
        Nat.0 < continued_fraction_convergent_denominator(coefficients, Nat.2 * k)
        continued_fraction_convergent_denominator_positive(
            coefficients, (Nat.2 * k).suc.suc)
        Nat.0 < continued_fraction_convergent_denominator(
            coefficients, (Nat.2 * k).suc.suc)
        nat_lt_imp_rat_lt(Nat.0, continued_fraction_convergent_denominator(
            coefficients, Nat.2 * k))
        Rat.from_nat(Nat.0) < Rat.from_nat(continued_fraction_convergent_denominator(
            coefficients, Nat.2 * k))
        Rat.from_nat(Nat.0) = Rat.0
        Rat.0 < Rat.from_nat(continued_fraction_convergent_denominator(
            coefficients, Nat.2 * k))
        zero_lt_imp_pos(Rat.from_nat(continued_fraction_convergent_denominator(
            coefficients, Nat.2 * k)))
        Rat.from_nat(continued_fraction_convergent_denominator(
            coefficients, Nat.2 * k)).is_positive
        nat_lt_imp_rat_lt(Nat.0, continued_fraction_convergent_denominator(
            coefficients, (Nat.2 * k).suc.suc))
        Rat.from_nat(Nat.0) < Rat.from_nat(continued_fraction_convergent_denominator(
            coefficients, (Nat.2 * k).suc.suc))
        Rat.from_nat(Nat.0) = Rat.0
        Rat.0 < Rat.from_nat(continued_fraction_convergent_denominator(
            coefficients, (Nat.2 * k).suc.suc))
        zero_lt_imp_pos(Rat.from_nat(continued_fraction_convergent_denominator(
            coefficients, (Nat.2 * k).suc.suc)))
        Rat.from_nat(continued_fraction_convergent_denominator(
            coefficients, (Nat.2 * k).suc.suc)).is_positive
        mul_pos_pos(
            Rat.from_nat(continued_fraction_convergent_denominator(
                coefficients, Nat.2 * k)),
            Rat.from_nat(continued_fraction_convergent_denominator(
                coefficients, (Nat.2 * k).suc.suc)))
        (Rat.from_nat(continued_fraction_convergent_denominator(
                coefficients, Nat.2 * k)) *
            Rat.from_nat(continued_fraction_convergent_denominator(
                coefficients, (Nat.2 * k).suc.suc))).is_positive
        rat_div_pos(
            Rat.from_nat(coefficients((Nat.2 * k).suc.suc)),
            Rat.from_nat(continued_fraction_convergent_denominator(
                coefficients, Nat.2 * k)) *
                Rat.from_nat(continued_fraction_convergent_denominator(
                    coefficients, (Nat.2 * k).suc.suc)))
        (Rat.from_nat(coefficients((Nat.2 * k).suc.suc)) /
            (Rat.from_nat(continued_fraction_convergent_denominator(
                    coefficients, Nat.2 * k)) *
                Rat.from_nat(continued_fraction_convergent_denominator(
                    coefficients, (Nat.2 * k).suc.suc)))).is_positive
        (continued_fraction_convergent_value(coefficients, (Nat.2 * k).suc.suc) -
            continued_fraction_convergent_value(coefficients, Nat.2 * k)).is_positive
        lt_add_pos(
            continued_fraction_convergent_value(coefficients, Nat.2 * k),
            continued_fraction_convergent_value(coefficients, (Nat.2 * k).suc.suc) -
                continued_fraction_convergent_value(coefficients, Nat.2 * k))
        continued_fraction_convergent_value(coefficients, Nat.2 * k) < continued_fraction_convergent_value(coefficients, Nat.2 * k) + (continued_fraction_convergent_value(coefficients, (Nat.2 * k).suc.suc) - continued_fraction_convergent_value(coefficients, Nat.2 * k))
        continued_fraction_convergent_value(coefficients, Nat.2 * k) +
            (continued_fraction_convergent_value(coefficients, (Nat.2 * k).suc.suc) -
                continued_fraction_convergent_value(coefficients, Nat.2 * k)) =
            continued_fraction_convergent_value(coefficients, (Nat.2 * k).suc.suc)
        continued_fraction_convergent_value(coefficients, Nat.2 * k) < continued_fraction_convergent_value(coefficients, (Nat.2 * k).suc.suc)
    }
}

/// At an odd index, the convergent two steps ahead is strictly smaller.
theorem continued_fraction_odd_convergent_value_two_suc_lt(
    coefficients: Nat -> Nat, k: Nat
) {
    positive_continued_fraction_sequence_tail(coefficients) implies continued_fraction_convergent_value(coefficients, (Nat.2 * k).suc.suc.suc) < continued_fraction_convergent_value(coefficients, (Nat.2 * k).suc)
} by {
    if positive_continued_fraction_sequence_tail(coefficients) {
        continued_fraction_alternating_sign_double_suc(k)
        alternating_sign[Int]((Nat.2 * k).suc) = -Int.1
        continued_fraction_numerator_difference_of_negative_sign(
            coefficients, (Nat.2 * k).suc)
        Rat.from_nat(continued_fraction_convergent_numerator(
                coefficients, (Nat.2 * k).suc.suc)) *
                Rat.from_nat(continued_fraction_convergent_denominator(
                    coefficients, (Nat.2 * k).suc)) -
            Rat.from_nat(continued_fraction_convergent_numerator(
                coefficients, (Nat.2 * k).suc)) *
                Rat.from_nat(continued_fraction_convergent_denominator(
                    coefficients, (Nat.2 * k).suc.suc)) =
            -Rat.1
        continued_fraction_convergent_value_two_suc_sub(coefficients, (Nat.2 * k).suc)
        continued_fraction_convergent_value(coefficients, (Nat.2 * k).suc.suc.suc) -
                continued_fraction_convergent_value(coefficients, (Nat.2 * k).suc) =
            (Rat.from_nat(coefficients((Nat.2 * k).suc.suc.suc)) *
                (Rat.from_nat(continued_fraction_convergent_numerator(
                        coefficients, (Nat.2 * k).suc.suc)) *
                    Rat.from_nat(continued_fraction_convergent_denominator(
                        coefficients, (Nat.2 * k).suc)) -
                    Rat.from_nat(continued_fraction_convergent_numerator(
                        coefficients, (Nat.2 * k).suc)) *
                    Rat.from_nat(continued_fraction_convergent_denominator(
                        coefficients, (Nat.2 * k).suc.suc)))) /
            (Rat.from_nat(continued_fraction_convergent_denominator(
                    coefficients, (Nat.2 * k).suc)) *
                Rat.from_nat(continued_fraction_convergent_denominator(
                    coefficients, (Nat.2 * k).suc.suc.suc)))
        continued_fraction_convergent_value(coefficients, (Nat.2 * k).suc.suc.suc) -
                continued_fraction_convergent_value(coefficients, (Nat.2 * k).suc) =
            (Rat.from_nat(coefficients((Nat.2 * k).suc.suc.suc)) * -Rat.1) /
            (Rat.from_nat(continued_fraction_convergent_denominator(
                    coefficients, (Nat.2 * k).suc)) *
                Rat.from_nat(continued_fraction_convergent_denominator(
                    coefficients, (Nat.2 * k).suc.suc.suc)))
        continued_fraction_convergent_value(coefficients, (Nat.2 * k).suc.suc.suc) -
                continued_fraction_convergent_value(coefficients, (Nat.2 * k).suc) =
            -(Rat.from_nat(coefficients((Nat.2 * k).suc.suc.suc)) /
                (Rat.from_nat(continued_fraction_convergent_denominator(
                        coefficients, (Nat.2 * k).suc)) *
                    Rat.from_nat(continued_fraction_convergent_denominator(
                        coefficients, (Nat.2 * k).suc.suc.suc))))
        Nat.0 < coefficients((Nat.2 * k).suc.suc.suc)
        nat_lt_imp_rat_lt(Nat.0, coefficients((Nat.2 * k).suc.suc.suc))
        Rat.from_nat(Nat.0) < Rat.from_nat(coefficients((Nat.2 * k).suc.suc.suc))
        from_nat_zero[Rat]
        Rat.from_nat(Nat.0) = Rat.0
        Rat.0 < Rat.from_nat(coefficients((Nat.2 * k).suc.suc.suc))
        zero_lt_imp_pos(Rat.from_nat(coefficients((Nat.2 * k).suc.suc.suc)))
        Rat.from_nat(coefficients((Nat.2 * k).suc.suc.suc)).is_positive
        continued_fraction_convergent_denominator_positive(
            coefficients, (Nat.2 * k).suc)
        continued_fraction_convergent_denominator_positive(
            coefficients, (Nat.2 * k).suc.suc.suc)
        nat_lt_imp_rat_lt(Nat.0, continued_fraction_convergent_denominator(
            coefficients, (Nat.2 * k).suc))
        Rat.from_nat(Nat.0) < Rat.from_nat(continued_fraction_convergent_denominator(
            coefficients, (Nat.2 * k).suc))
        Rat.from_nat(Nat.0) = Rat.0
        Rat.0 < Rat.from_nat(continued_fraction_convergent_denominator(
            coefficients, (Nat.2 * k).suc))
        zero_lt_imp_pos(Rat.from_nat(continued_fraction_convergent_denominator(
            coefficients, (Nat.2 * k).suc)))
        Rat.from_nat(continued_fraction_convergent_denominator(
            coefficients, (Nat.2 * k).suc)).is_positive
        nat_lt_imp_rat_lt(Nat.0, continued_fraction_convergent_denominator(
            coefficients, (Nat.2 * k).suc.suc.suc))
        Rat.from_nat(Nat.0) < Rat.from_nat(continued_fraction_convergent_denominator(
            coefficients, (Nat.2 * k).suc.suc.suc))
        Rat.from_nat(Nat.0) = Rat.0
        Rat.0 < Rat.from_nat(continued_fraction_convergent_denominator(
            coefficients, (Nat.2 * k).suc.suc.suc))
        zero_lt_imp_pos(Rat.from_nat(continued_fraction_convergent_denominator(
            coefficients, (Nat.2 * k).suc.suc.suc)))
        Rat.from_nat(continued_fraction_convergent_denominator(
            coefficients, (Nat.2 * k).suc.suc.suc)).is_positive
        mul_pos_pos(
            Rat.from_nat(continued_fraction_convergent_denominator(
                coefficients, (Nat.2 * k).suc)),
            Rat.from_nat(continued_fraction_convergent_denominator(
                coefficients, (Nat.2 * k).suc.suc.suc)))
        (Rat.from_nat(continued_fraction_convergent_denominator(
                coefficients, (Nat.2 * k).suc)) *
            Rat.from_nat(continued_fraction_convergent_denominator(
                coefficients, (Nat.2 * k).suc.suc.suc))).is_positive
        rat_div_pos(
            Rat.from_nat(coefficients((Nat.2 * k).suc.suc.suc)),
            Rat.from_nat(continued_fraction_convergent_denominator(
                coefficients, (Nat.2 * k).suc)) *
                Rat.from_nat(continued_fraction_convergent_denominator(
                    coefficients, (Nat.2 * k).suc.suc.suc)))
        (Rat.from_nat(coefficients((Nat.2 * k).suc.suc.suc)) /
            (Rat.from_nat(continued_fraction_convergent_denominator(
                    coefficients, (Nat.2 * k).suc)) *
                Rat.from_nat(continued_fraction_convergent_denominator(
                    coefficients, (Nat.2 * k).suc.suc.suc)))).is_positive
        neg_pos_is_neg(
            Rat.from_nat(coefficients((Nat.2 * k).suc.suc.suc)) /
            (Rat.from_nat(continued_fraction_convergent_denominator(
                    coefficients, (Nat.2 * k).suc)) *
                Rat.from_nat(continued_fraction_convergent_denominator(
                    coefficients, (Nat.2 * k).suc.suc.suc))))
        (continued_fraction_convergent_value(coefficients, (Nat.2 * k).suc.suc.suc) -
            continued_fraction_convergent_value(coefficients, (Nat.2 * k).suc)) .is_negative
        neg_neg_is_pos(
            continued_fraction_convergent_value(coefficients, (Nat.2 * k).suc.suc.suc) -
            continued_fraction_convergent_value(coefficients, (Nat.2 * k).suc))
        (-(continued_fraction_convergent_value(coefficients, (Nat.2 * k).suc.suc.suc) -
            continued_fraction_convergent_value(coefficients, (Nat.2 * k).suc))) .is_positive
        lt_add_pos(
            continued_fraction_convergent_value(coefficients, (Nat.2 * k).suc.suc.suc),
            -(continued_fraction_convergent_value(coefficients, (Nat.2 * k).suc.suc.suc) -
                continued_fraction_convergent_value(coefficients, (Nat.2 * k).suc)))
        continued_fraction_convergent_value(coefficients, (Nat.2 * k).suc.suc.suc) < continued_fraction_convergent_value(coefficients, (Nat.2 * k).suc.suc.suc) + (-(continued_fraction_convergent_value(coefficients, (Nat.2 * k).suc.suc.suc) - continued_fraction_convergent_value(coefficients, (Nat.2 * k).suc)))
        -(continued_fraction_convergent_value(coefficients, (Nat.2 * k).suc.suc.suc) -
            continued_fraction_convergent_value(coefficients, (Nat.2 * k).suc)) =
            continued_fraction_convergent_value(coefficients, (Nat.2 * k).suc) -
            continued_fraction_convergent_value(coefficients, (Nat.2 * k).suc.suc.suc)
        continued_fraction_convergent_value(coefficients, (Nat.2 * k).suc.suc.suc) +
            (-(continued_fraction_convergent_value(coefficients, (Nat.2 * k).suc.suc.suc) -
                continued_fraction_convergent_value(coefficients, (Nat.2 * k).suc))) =
            continued_fraction_convergent_value(coefficients, (Nat.2 * k).suc.suc.suc) +
            (continued_fraction_convergent_value(coefficients, (Nat.2 * k).suc) -
                continued_fraction_convergent_value(coefficients, (Nat.2 * k).suc.suc.suc))
        continued_fraction_convergent_value(coefficients, (Nat.2 * k).suc.suc.suc) +
            (continued_fraction_convergent_value(coefficients, (Nat.2 * k).suc) -
                continued_fraction_convergent_value(coefficients, (Nat.2 * k).suc.suc.suc)) =
            continued_fraction_convergent_value(coefficients, (Nat.2 * k).suc)
        continued_fraction_convergent_value(coefficients, (Nat.2 * k).suc.suc.suc) +
            (-(continued_fraction_convergent_value(coefficients, (Nat.2 * k).suc.suc.suc) -
                continued_fraction_convergent_value(coefficients, (Nat.2 * k).suc))) =
            continued_fraction_convergent_value(coefficients, (Nat.2 * k).suc)
        continued_fraction_convergent_value(coefficients, (Nat.2 * k).suc.suc.suc) < continued_fraction_convergent_value(coefficients, (Nat.2 * k).suc)
    }
}

/// Doubling distributes over a successor index.
theorem continued_fraction_double_suc_suc(k: Nat) {
    Nat.2 * k.suc = (Nat.2 * k).suc.suc
} by {
    mul_suc_right(Nat.2, k)
    Nat.2 * k.suc = Nat.2 + Nat.2 * k
    add_comm(Nat.2, Nat.2 * k)
    Nat.2 + Nat.2 * k = Nat.2 * k + Nat.2
    add_suc_right(Nat.2 * k, Nat.1)
    Nat.2 * k + Nat.2 = (Nat.2 * k + Nat.1).suc
    add_one_right(Nat.2 * k)
    Nat.2 * k + Nat.1 = (Nat.2 * k).suc
    Nat.2 * k + Nat.2 = (Nat.2 * k).suc.suc
    Nat.2 * k.suc = (Nat.2 * k).suc.suc
}

/// Even-indexed convergent values are nondecreasing in the index.
theorem continued_fraction_even_value_lte_distant(coefficients: Nat -> Nat, k: Nat, l: Nat) {
    positive_continued_fraction_sequence_tail(coefficients) and k <= l implies continued_fraction_convergent_value(coefficients, Nat.2 * k) <= continued_fraction_convergent_value(coefficients, Nat.2 * l)
} by {
    if positive_continued_fraction_sequence_tail(coefficients) and k <= l {
        define q(d: Nat) -> Bool {
            forall(k1: Nat, l1: Nat) {
                k1 + d = l1 implies continued_fraction_convergent_value(coefficients, Nat.2 * k1) <= continued_fraction_convergent_value(coefficients, Nat.2 * l1)
            }
        }
        forall(k1: Nat, l1: Nat) {
            if k1 + Nat.0 = l1 {
                k1 + Nat.0 = k1
                continued_fraction_convergent_value(coefficients, Nat.2 * l1) <= continued_fraction_convergent_value(coefficients, Nat.2 * l1)
                continued_fraction_convergent_value(coefficients, Nat.2 * k1) <= continued_fraction_convergent_value(coefficients, Nat.2 * l1)
            }
        }
        q(Nat.0)
        forall(d: Nat) {
            if q(d) {
                forall(k1: Nat, l1: Nat) {
                    if k1 + d.suc = l1 {
                        add_suc_right(k1, d)
                        k1 + d.suc = (k1 + d).suc
                        (k1 + d).suc = l1
                        k1 + d = k1 + d
                        continued_fraction_convergent_value(coefficients, Nat.2 * k1) <= continued_fraction_convergent_value(coefficients, Nat.2 * (k1 + d))
                        continued_fraction_even_convergent_value_two_suc_gt(
                            coefficients, k1 + d)
                        continued_fraction_convergent_value(
                                coefficients, Nat.2 * (k1 + d)) < continued_fraction_convergent_value(
                                coefficients, (Nat.2 * (k1 + d)).suc.suc)
                        continued_fraction_double_suc_suc(k1 + d)
                        Nat.2 * (k1 + d).suc = (Nat.2 * (k1 + d)).suc.suc
                        (k1 + d).suc = l1
                        Nat.2 * l1 = Nat.2 * (k1 + d).suc
                        continued_fraction_convergent_value(
                                coefficients, (Nat.2 * (k1 + d)).suc.suc) =
                            continued_fraction_convergent_value(
                                coefficients, Nat.2 * l1)
                        continued_fraction_convergent_value(
                                coefficients, Nat.2 * (k1 + d)) < continued_fraction_convergent_value(
                                coefficients, Nat.2 * l1)
                        lt_of_lte_of_lt(
                            continued_fraction_convergent_value(coefficients, Nat.2 * k1),
                            continued_fraction_convergent_value(
                                coefficients, Nat.2 * (k1 + d)),
                            continued_fraction_convergent_value(
                                coefficients, Nat.2 * l1))
                        continued_fraction_convergent_value(coefficients, Nat.2 * k1) <= continued_fraction_convergent_value(coefficients, Nat.2 * l1)
                    }
                }
                q(d.suc)
            }
        }
        q(Nat.0) and forall(d: Nat) { q(d) implies q(d.suc) }
        Nat.induction(q)
        forall(d: Nat) { q(d) }
        let d: Nat satisfy {
            k + d = l
        }
        q(d)
        k + d = l implies continued_fraction_convergent_value(coefficients, Nat.2 * k) <= continued_fraction_convergent_value(coefficients, Nat.2 * l)
        continued_fraction_convergent_value(coefficients, Nat.2 * k) <= continued_fraction_convergent_value(coefficients, Nat.2 * l)
    }
}

/// Odd-indexed convergent values are nonincreasing in the index.
theorem continued_fraction_odd_value_lte_distant(coefficients: Nat -> Nat, k: Nat, l: Nat) {
    positive_continued_fraction_sequence_tail(coefficients) and k <= l implies continued_fraction_convergent_value(coefficients, (Nat.2 * l).suc) <= continued_fraction_convergent_value(coefficients, (Nat.2 * k).suc)
} by {
    if positive_continued_fraction_sequence_tail(coefficients) and k <= l {
        define q(d: Nat) -> Bool {
            forall(k1: Nat, l1: Nat) {
                k1 + d = l1 implies continued_fraction_convergent_value(coefficients, (Nat.2 * l1).suc) <= continued_fraction_convergent_value(coefficients, (Nat.2 * k1).suc)
            }
        }
        forall(k1: Nat, l1: Nat) {
            if k1 + Nat.0 = l1 {
                k1 + Nat.0 = k1
                continued_fraction_convergent_value(coefficients, (Nat.2 * l1).suc) <= continued_fraction_convergent_value(coefficients, (Nat.2 * l1).suc)
                continued_fraction_convergent_value(coefficients, (Nat.2 * l1).suc) <= continued_fraction_convergent_value(coefficients, (Nat.2 * k1).suc)
            }
        }
        q(Nat.0)
        forall(d: Nat) {
            if q(d) {
                forall(k1: Nat, l1: Nat) {
                    if k1 + d.suc = l1 {
                        add_suc_right(k1, d)
                        k1 + d.suc = (k1 + d).suc
                        (k1 + d).suc = l1
                        continued_fraction_convergent_value(coefficients, (Nat.2 * (k1 + d)).suc) <= continued_fraction_convergent_value(coefficients, (Nat.2 * k1).suc)
                        continued_fraction_odd_convergent_value_two_suc_lt(
                            coefficients, k1 + d)
                        continued_fraction_convergent_value(
                                coefficients, (Nat.2 * (k1 + d)).suc.suc.suc) < continued_fraction_convergent_value(
                                coefficients, (Nat.2 * (k1 + d)).suc)
                        continued_fraction_double_suc_suc(k1 + d)
                        Nat.2 * (k1 + d).suc = (Nat.2 * (k1 + d)).suc.suc
                        (k1 + d).suc = l1
                        Nat.2 * l1 = Nat.2 * (k1 + d).suc
                        (Nat.2 * l1).suc = (Nat.2 * (k1 + d).suc).suc
                        (Nat.2 * (k1 + d)).suc.suc.suc = (Nat.2 * (k1 + d).suc).suc
                        continued_fraction_convergent_value(
                                coefficients, (Nat.2 * (k1 + d)).suc.suc.suc) =
                            continued_fraction_convergent_value(
                                coefficients, (Nat.2 * l1).suc)
                        continued_fraction_convergent_value(
                                coefficients, (Nat.2 * l1).suc) < continued_fraction_convergent_value(
                                coefficients, (Nat.2 * (k1 + d)).suc)
                        lt_of_lt_of_lte(
                            continued_fraction_convergent_value(
                                coefficients, (Nat.2 * l1).suc),
                            continued_fraction_convergent_value(
                                coefficients, (Nat.2 * (k1 + d)).suc),
                            continued_fraction_convergent_value(
                                coefficients, (Nat.2 * k1).suc))
                        continued_fraction_convergent_value(coefficients, (Nat.2 * l1).suc) <= continued_fraction_convergent_value(coefficients, (Nat.2 * k1).suc)
                    }
                }
                q(d.suc)
            }
        }
        q(Nat.0) and forall(d: Nat) { q(d) implies q(d.suc) }
        Nat.induction(q)
        forall(d: Nat) { q(d) }
        let d: Nat satisfy {
            k + d = l
        }
        q(d)
        k + d = l implies continued_fraction_convergent_value(coefficients, (Nat.2 * l).suc) <= continued_fraction_convergent_value(coefficients, (Nat.2 * k).suc)
        continued_fraction_convergent_value(coefficients, (Nat.2 * l).suc) <= continued_fraction_convergent_value(coefficients, (Nat.2 * k).suc)
    }
}

/// Every even-indexed convergent value is below every odd-indexed one.
theorem continued_fraction_even_lte_odd(coefficients: Nat -> Nat, k: Nat, l: Nat) {
    positive_continued_fraction_sequence_tail(coefficients) implies continued_fraction_convergent_value(coefficients, Nat.2 * k) <= continued_fraction_convergent_value(coefficients, (Nat.2 * l).suc)
} by {
    if positive_continued_fraction_sequence_tail(coefficients) {
        lt_or_lte(l, k)
        l < k or k <= l
        if l < k {
            continued_fraction_odd_value_lte_distant(coefficients, l, k)
            continued_fraction_convergent_value(coefficients, (Nat.2 * k).suc) <= continued_fraction_convergent_value(coefficients, (Nat.2 * l).suc)
            continued_fraction_even_convergent_lt_following_odd(coefficients, k)
            continued_fraction_convergent_value(coefficients, Nat.2 * k) < continued_fraction_convergent_value(coefficients, (Nat.2 * k).suc)
            lt_of_lt_of_lte(
                continued_fraction_convergent_value(coefficients, Nat.2 * k),
                continued_fraction_convergent_value(coefficients, (Nat.2 * k).suc),
                continued_fraction_convergent_value(coefficients, (Nat.2 * l).suc))
            continued_fraction_convergent_value(coefficients, Nat.2 * k) <= continued_fraction_convergent_value(coefficients, (Nat.2 * l).suc)
            l < k implies continued_fraction_convergent_value(coefficients, Nat.2 * k) <= continued_fraction_convergent_value(coefficients, (Nat.2 * l).suc)
        }
        if k <= l {
            continued_fraction_even_value_lte_distant(coefficients, k, l)
            continued_fraction_convergent_value(coefficients, Nat.2 * k) <= continued_fraction_convergent_value(coefficients, Nat.2 * l)
            continued_fraction_even_convergent_lt_following_odd(coefficients, l)
            continued_fraction_convergent_value(coefficients, Nat.2 * l) < continued_fraction_convergent_value(coefficients, (Nat.2 * l).suc)
            lt_of_lte_of_lt(
                continued_fraction_convergent_value(coefficients, Nat.2 * k),
                continued_fraction_convergent_value(coefficients, Nat.2 * l),
                continued_fraction_convergent_value(coefficients, (Nat.2 * l).suc))
            continued_fraction_convergent_value(coefficients, Nat.2 * k) <= continued_fraction_convergent_value(coefficients, (Nat.2 * l).suc)
            k <= l implies continued_fraction_convergent_value(coefficients, Nat.2 * k) <= continued_fraction_convergent_value(coefficients, (Nat.2 * l).suc)
        }
        continued_fraction_convergent_value(coefficients, Nat.2 * k) <= continued_fraction_convergent_value(coefficients, (Nat.2 * l).suc)
    }
}

/// The real even-indexed convergent subsequence.
define continued_fraction_real_even_convergent(
    coefficients: Nat -> Nat, k: Nat
) -> Real {
    continued_fraction_real_convergent_value(coefficients, Nat.2 * k)
}

/// The real odd-indexed convergent subsequence.
define continued_fraction_real_odd_convergent(
    coefficients: Nat -> Nat, k: Nat
) -> Real {
    continued_fraction_real_convergent_value(coefficients, (Nat.2 * k).suc)
}

/// The real limit of the convergent sequence.
define continued_fraction_real_limit(coefficients: Nat -> Nat) -> Real {
    limit(continued_fraction_real_even_convergent(coefficients))
}

/// The real even-indexed convergent subsequence is increasing.
theorem continued_fraction_real_even_increasing(coefficients: Nat -> Nat) {
    positive_continued_fraction_sequence_tail(coefficients) implies is_increasing(continued_fraction_real_even_convergent(coefficients))
} by {
    if positive_continued_fraction_sequence_tail(coefficients) {
        forall(k: Nat) {
            continued_fraction_even_convergent_value_two_suc_gt(coefficients, k)
            continued_fraction_convergent_value(coefficients, Nat.2 * k) < continued_fraction_convergent_value(coefficients, (Nat.2 * k).suc.suc)
            lt_imp_lte(
                continued_fraction_convergent_value(coefficients, Nat.2 * k),
                continued_fraction_convergent_value(coefficients, (Nat.2 * k).suc.suc))
            continued_fraction_convergent_value(coefficients, Nat.2 * k) <= continued_fraction_convergent_value(coefficients, (Nat.2 * k).suc.suc)
            from_rat_maintains_lte(
                continued_fraction_convergent_value(coefficients, Nat.2 * k),
                continued_fraction_convergent_value(coefficients, (Nat.2 * k).suc.suc))
            Real.from_rat(continued_fraction_convergent_value(coefficients, Nat.2 * k)) <= Real.from_rat(continued_fraction_convergent_value(
                    coefficients, (Nat.2 * k).suc.suc))
            continued_fraction_real_convergent_value(coefficients, Nat.2 * k) <= continued_fraction_real_convergent_value(
                    coefficients, (Nat.2 * k).suc.suc)
            continued_fraction_double_suc_suc(k)
            Nat.2 * k.suc = (Nat.2 * k).suc.suc
            continued_fraction_real_convergent_value(
                    coefficients, (Nat.2 * k).suc.suc) =
                continued_fraction_real_convergent_value(
                    coefficients, Nat.2 * k.suc)
            continued_fraction_real_convergent_value(coefficients, Nat.2 * k) <= continued_fraction_real_convergent_value(coefficients, Nat.2 * k.suc)
            continued_fraction_real_even_convergent(coefficients, k) <= continued_fraction_real_even_convergent(coefficients, k.suc)
        }
        is_increasing(continued_fraction_real_even_convergent(coefficients))
    }
}

/// The real even-indexed convergent subsequence is bounded above by the first
/// odd convergent.
theorem continued_fraction_real_even_bounded_above(coefficients: Nat -> Nat) {
    positive_continued_fraction_sequence_tail(coefficients) implies is_upper_bound(continued_fraction_real_even_convergent(coefficients), continued_fraction_real_convergent_value(coefficients, Nat.1))
} by {
    if positive_continued_fraction_sequence_tail(coefficients) {
        forall(k: Nat) {
            continued_fraction_even_lte_odd(coefficients, k, Nat.0)
            continued_fraction_convergent_value(coefficients, Nat.2 * k) <= continued_fraction_convergent_value(coefficients, (Nat.2 * Nat.0).suc)
            mul_zero_right(Nat.2)
            Nat.2 * Nat.0 = Nat.0
            continued_fraction_convergent_value(coefficients, Nat.2 * k) <= continued_fraction_convergent_value(coefficients, Nat.1)
            from_rat_maintains_lte(
                continued_fraction_convergent_value(coefficients, Nat.2 * k),
                continued_fraction_convergent_value(coefficients, Nat.1))
            Real.from_rat(continued_fraction_convergent_value(coefficients, Nat.2 * k)) <= Real.from_rat(continued_fraction_convergent_value(coefficients, Nat.1))
            continued_fraction_real_convergent_value(coefficients, Nat.2 * k) <= continued_fraction_real_convergent_value(coefficients, Nat.1)
            continued_fraction_real_even_convergent(coefficients, k) <= continued_fraction_real_convergent_value(coefficients, Nat.1)
        }
        is_upper_bound(continued_fraction_real_even_convergent(coefficients),
            continued_fraction_real_convergent_value(coefficients, Nat.1))
    }
}

/// The real even-indexed convergent subsequence converges.
theorem continued_fraction_real_even_converges(coefficients: Nat -> Nat) {
    positive_continued_fraction_sequence_tail(coefficients) implies converges(continued_fraction_real_even_convergent(coefficients))
} by {
    if positive_continued_fraction_sequence_tail(coefficients) {
        continued_fraction_real_even_increasing(coefficients)
        is_increasing(continued_fraction_real_even_convergent(coefficients))
        continued_fraction_real_even_bounded_above(coefficients)
        is_upper_bound(continued_fraction_real_even_convergent(coefficients),
            continued_fraction_real_convergent_value(coefficients, Nat.1))
        monotone_convergence_principle(
            continued_fraction_real_even_convergent(coefficients),
            continued_fraction_real_convergent_value(coefficients, Nat.1))
        converges(continued_fraction_real_even_convergent(coefficients))
    }
}

/// Every even-indexed convergent value is at most the real limit.
theorem continued_fraction_real_even_le_limit(coefficients: Nat -> Nat, k: Nat) {
    positive_continued_fraction_sequence_tail(coefficients) implies continued_fraction_real_even_convergent(coefficients, k) <= continued_fraction_real_limit(coefficients)
} by {
    if positive_continued_fraction_sequence_tail(coefficients) {
        continued_fraction_real_even_increasing(coefficients)
        is_increasing(continued_fraction_real_even_convergent(coefficients))
        continued_fraction_real_even_converges(coefficients)
        converges(continued_fraction_real_even_convergent(coefficients))
        increasing_convergent_bounded_by_limit(
            continued_fraction_real_even_convergent(coefficients))
        is_upper_bound(continued_fraction_real_even_convergent(coefficients),
            limit(continued_fraction_real_even_convergent(coefficients)))
        continued_fraction_real_even_convergent(coefficients, k) <= limit(continued_fraction_real_even_convergent(coefficients))
        continued_fraction_real_even_convergent(coefficients, k) <= continued_fraction_real_limit(coefficients)
    }
}

/// Every odd-indexed convergent value is at least the real limit.
theorem continued_fraction_real_limit_le_odd(coefficients: Nat -> Nat, l: Nat) {
    positive_continued_fraction_sequence_tail(coefficients) implies continued_fraction_real_limit(coefficients) <= continued_fraction_real_odd_convergent(coefficients, l)
} by {
    if positive_continued_fraction_sequence_tail(coefficients) {
        forall(k: Nat) {
            continued_fraction_even_lte_odd(coefficients, k, l)
            continued_fraction_convergent_value(coefficients, Nat.2 * k) <= continued_fraction_convergent_value(coefficients, (Nat.2 * l).suc)
            from_rat_maintains_lte(
                continued_fraction_convergent_value(coefficients, Nat.2 * k),
                continued_fraction_convergent_value(coefficients, (Nat.2 * l).suc))
            Real.from_rat(continued_fraction_convergent_value(coefficients, Nat.2 * k)) <= Real.from_rat(continued_fraction_convergent_value(
                    coefficients, (Nat.2 * l).suc))
            continued_fraction_real_convergent_value(coefficients, Nat.2 * k) <= continued_fraction_real_convergent_value(
                    coefficients, (Nat.2 * l).suc)
            continued_fraction_real_even_convergent(coefficients, k) <= continued_fraction_real_odd_convergent(coefficients, l)
        }
        seq_lte(
            continued_fraction_real_even_convergent(coefficients),
            constant[Nat, Real](continued_fraction_real_odd_convergent(coefficients, l)))
        continued_fraction_real_even_converges(coefficients)
        converges(continued_fraction_real_even_convergent(coefficients))
        const_converges(continued_fraction_real_odd_convergent(coefficients, l))
        converges(constant[Nat, Real](continued_fraction_real_odd_convergent(coefficients, l)))
        seq_lte_preserves_limit(
            continued_fraction_real_even_convergent(coefficients),
            constant[Nat, Real](continued_fraction_real_odd_convergent(coefficients, l)))
        limit(continued_fraction_real_even_convergent(coefficients)) <= limit(constant[Nat, Real](continued_fraction_real_odd_convergent(coefficients, l)))
        const_limit(continued_fraction_real_odd_convergent(coefficients, l))
        limit(constant[Nat, Real](continued_fraction_real_odd_convergent(coefficients, l))) =
            continued_fraction_real_odd_convergent(coefficients, l)
        limit(continued_fraction_real_even_convergent(coefficients)) <= continued_fraction_real_odd_convergent(coefficients, l)
        continued_fraction_real_limit(coefficients) <= continued_fraction_real_odd_convergent(coefficients, l)
    }
}

/// Every even-indexed convergent value is strictly below the real limit.
theorem continued_fraction_real_even_lt_limit(coefficients: Nat -> Nat, k: Nat) {
    positive_continued_fraction_sequence_tail(coefficients) implies continued_fraction_real_even_convergent(coefficients, k) < continued_fraction_real_limit(coefficients)
} by {
    if positive_continued_fraction_sequence_tail(coefficients) {
        continued_fraction_even_convergent_value_two_suc_gt(coefficients, k)
        continued_fraction_convergent_value(coefficients, Nat.2 * k) < continued_fraction_convergent_value(coefficients, (Nat.2 * k).suc.suc)
        from_rat_maintains_lt(
            continued_fraction_convergent_value(coefficients, Nat.2 * k),
            continued_fraction_convergent_value(coefficients, (Nat.2 * k).suc.suc))
        Real.from_rat(continued_fraction_convergent_value(coefficients, Nat.2 * k)) < Real.from_rat(continued_fraction_convergent_value(
                coefficients, (Nat.2 * k).suc.suc))
        continued_fraction_real_convergent_value(coefficients, Nat.2 * k) < continued_fraction_real_convergent_value(coefficients, (Nat.2 * k).suc.suc)
        continued_fraction_double_suc_suc(k)
        Nat.2 * k.suc = (Nat.2 * k).suc.suc
        continued_fraction_real_convergent_value(coefficients, (Nat.2 * k).suc.suc) =
            continued_fraction_real_convergent_value(coefficients, Nat.2 * k.suc)
        continued_fraction_real_convergent_value(coefficients, Nat.2 * k) < continued_fraction_real_convergent_value(coefficients, Nat.2 * k.suc)
        continued_fraction_real_even_convergent(coefficients, k) < continued_fraction_real_even_convergent(coefficients, k.suc)
        continued_fraction_real_even_le_limit(coefficients, k.suc)
        continued_fraction_real_even_convergent(coefficients, k.suc) <= continued_fraction_real_limit(coefficients)
        lt_of_lt_of_lte(
            continued_fraction_real_even_convergent(coefficients, k),
            continued_fraction_real_even_convergent(coefficients, k.suc),
            continued_fraction_real_limit(coefficients))
        continued_fraction_real_even_convergent(coefficients, k) < continued_fraction_real_limit(coefficients)
    }
}

/// The real limit is strictly below every odd-indexed convergent value.
theorem continued_fraction_real_limit_lt_odd(coefficients: Nat -> Nat, l: Nat) {
    positive_continued_fraction_sequence_tail(coefficients) implies continued_fraction_real_limit(coefficients) < continued_fraction_real_odd_convergent(coefficients, l)
} by {
    if positive_continued_fraction_sequence_tail(coefficients) {
        continued_fraction_odd_convergent_value_two_suc_lt(coefficients, l)
        continued_fraction_convergent_value(coefficients, (Nat.2 * l).suc.suc.suc) < continued_fraction_convergent_value(coefficients, (Nat.2 * l).suc)
        from_rat_maintains_lt(
            continued_fraction_convergent_value(coefficients, (Nat.2 * l).suc.suc.suc),
            continued_fraction_convergent_value(coefficients, (Nat.2 * l).suc))
        Real.from_rat(continued_fraction_convergent_value(
                coefficients, (Nat.2 * l).suc.suc.suc)) < Real.from_rat(continued_fraction_convergent_value(
                coefficients, (Nat.2 * l).suc))
        continued_fraction_real_convergent_value(
                coefficients, (Nat.2 * l).suc.suc.suc) < continued_fraction_real_convergent_value(
                coefficients, (Nat.2 * l).suc)
        continued_fraction_double_suc_suc(l)
        Nat.2 * l.suc = (Nat.2 * l).suc.suc
        (Nat.2 * l.suc).suc = (Nat.2 * l).suc.suc.suc
        continued_fraction_real_convergent_value(
                coefficients, (Nat.2 * l).suc.suc.suc) =
            continued_fraction_real_convergent_value(
                coefficients, (Nat.2 * l.suc).suc)
        continued_fraction_real_odd_convergent(coefficients, l.suc) < continued_fraction_real_odd_convergent(coefficients, l)
        continued_fraction_real_limit_le_odd(coefficients, l.suc)
        continued_fraction_real_limit(coefficients) <= continued_fraction_real_odd_convergent(coefficients, l.suc)
        lt_of_lte_of_lt(
            continued_fraction_real_limit(coefficients),
            continued_fraction_real_odd_convergent(coefficients, l.suc),
            continued_fraction_real_odd_convergent(coefficients, l))
        continued_fraction_real_limit(coefficients) < continued_fraction_real_odd_convergent(coefficients, l)
    }
}

/// Every natural number is twice an index or one more than twice an index.
theorem nat_even_or_odd(n: Nat) {
    exists(k: Nat) { n = Nat.2 * k } or exists(k: Nat) { n = (Nat.2 * k).suc }
} by {
    define p(m: Nat) -> Bool {
        exists(k: Nat) { m = Nat.2 * k } or exists(k: Nat) { m = (Nat.2 * k).suc }
    }
    mul_zero_right(Nat.2)
    Nat.2 * Nat.0 = Nat.0
    p(Nat.0)
    forall(m: Nat) {
        if p(m) {
            if exists(k: Nat) { m = Nat.2 * k } {
                let k: Nat satisfy {
                    m = Nat.2 * k
                }
                m = Nat.2 * k
                m.suc = (Nat.2 * k).suc
                exists(k1: Nat) { m.suc = (Nat.2 * k1).suc }
                p(m.suc)
            }
            if exists(k: Nat) { m = (Nat.2 * k).suc } {
                let k: Nat satisfy {
                    m = (Nat.2 * k).suc
                }
                continued_fraction_double_suc_suc(k)
                Nat.2 * k.suc = (Nat.2 * k).suc.suc
                m.suc = (Nat.2 * k).suc.suc
                m.suc = Nat.2 * k.suc
                exists(k1: Nat) { m.suc = Nat.2 * k1 }
                p(m.suc)
            }
            p(m.suc)
        }
        p(m) implies p(m.suc)
    }
    p(Nat.0) and forall(m: Nat) { p(m) implies p(m.suc) }
    Nat.induction(p)
    forall(m: Nat) { p(m) }
    p(n)
}

/// A real strictly between two reals is strictly closer to the left one than
/// the right one is.
theorem real_strict_between_abs_lt(a: Real, e: Real, b: Real) {
    a < e and e < b implies (e - a).abs < (b - a).abs
} by {
    if a < e and e < b {
        lt_imp_minus_pos(a, e)
        (e - a).is_positive
        pos_imp_eq_abs(e - a)
        (e - a).abs = e - a
        lt_trans(a, e, b)
        a < b
        lt_imp_minus_pos(a, b)
        (b - a).is_positive
        pos_imp_eq_abs(b - a)
        (b - a).abs = b - a
        lt_add_right(e, b, -a)
        e + -a < b + -a
        e + -a = e - a
        b + -a = b - a
        e - a < b - a
        (e - a).abs < (b - a).abs
    }
}

/// The embedded product of two positive naturals is a positive real.
theorem real_from_nat_product_positive(a: Nat, b: Nat) {
    Nat.0 < a and Nat.0 < b implies Real.from_rat(Rat.from_nat(a * b)).is_positive
} by {
    if Nat.0 < a and Nat.0 < b {
        nat_mul_positive(a, b)
        Nat.0 < a * b
        nat_lt_imp_rat_lt(Nat.0, a * b)
        Rat.from_nat(Nat.0) < Rat.from_nat(a * b)
        from_nat_zero[Rat]
        Rat.from_nat(Nat.0) = Rat.0
        Rat.0 < Rat.from_nat(a * b)
        from_rat_maintains_lt(Rat.0, Rat.from_nat(a * b))
        Real.from_rat(Rat.0) < Real.from_rat(Rat.from_nat(a * b))
        Real.from_rat(Rat.0) = Real.0
        Real.0 < Real.from_rat(Rat.from_nat(a * b))
        gt_zero_imp_pos(Real.from_rat(Rat.from_nat(a * b)))
        Real.from_rat(Rat.from_nat(a * b)).is_positive
    }
}
/// The real reciprocal of an embedded natural is the embedded inverse.
theorem real_from_nat_recip_inverse(n: Nat) {
    Rat.from_nat(n) != Rat.0 implies Real.from_rat(Rat.1 / Rat.from_nat(n)) = Real.from_rat(Rat.from_nat(n)).inverse
} by {
    if Rat.from_nat(n) != Rat.0 {
        recip_eq_one_div(Rat.from_nat(n))
        Rat.from_nat(n).inverse = Rat.1 / Rat.from_nat(n)
        real_from_rat_inverse(Rat.from_nat(n))
        Real.from_rat(Rat.from_nat(n).inverse) = Real.from_rat(Rat.from_nat(n)).inverse
        Real.from_rat(Rat.1 / Rat.from_nat(n)) = Real.from_rat(Rat.from_nat(n)).inverse
    }
}

/// The real gap bound at a positive index is below the reciprocal of the
/// squared denominator.
theorem continued_fraction_real_gap_lt_square(coefficients: Nat -> Nat, m: Nat) {
    positive_continued_fraction_sequence_tail(coefficients) implies continued_fraction_real_gap_bound(coefficients, m.suc) < Real.from_rat(Rat.1 / Rat.from_nat(
            continued_fraction_convergent_denominator(coefficients, m.suc) *
            continued_fraction_convergent_denominator(coefficients, m.suc)))
} by {
    if positive_continued_fraction_sequence_tail(coefficients) {
        continued_fraction_convergent_denominator_suc_gt(coefficients, m)
        continued_fraction_convergent_denominator(coefficients, m.suc) < continued_fraction_convergent_denominator(coefficients, m.suc.suc)
        continued_fraction_convergent_denominator_positive(coefficients, m.suc)
        Nat.0 < continued_fraction_convergent_denominator(coefficients, m.suc)
        nat_positive_ne_zero(continued_fraction_convergent_denominator(coefficients, m.suc))
        continued_fraction_convergent_denominator(coefficients, m.suc) != Nat.0
        lt_mul_both(
            continued_fraction_convergent_denominator(coefficients, m.suc),
            continued_fraction_convergent_denominator(coefficients, m.suc),
            continued_fraction_convergent_denominator(coefficients, m.suc.suc))
        continued_fraction_convergent_denominator(coefficients, m.suc) *
                continued_fraction_convergent_denominator(coefficients, m.suc) < continued_fraction_convergent_denominator(coefficients, m.suc) * continued_fraction_convergent_denominator(coefficients, m.suc.suc)
        nat_lt_imp_rat_lt(
            continued_fraction_convergent_denominator(coefficients, m.suc) *
                continued_fraction_convergent_denominator(coefficients, m.suc),
            continued_fraction_convergent_denominator(coefficients, m.suc) *
                continued_fraction_convergent_denominator(coefficients, m.suc.suc))
        Rat.from_nat(continued_fraction_convergent_denominator(coefficients, m.suc) *
                continued_fraction_convergent_denominator(coefficients, m.suc)) < Rat.from_nat(continued_fraction_convergent_denominator(coefficients, m.suc) * continued_fraction_convergent_denominator(coefficients, m.suc.suc))
        from_rat_maintains_lt(
            Rat.from_nat(continued_fraction_convergent_denominator(coefficients, m.suc) *
                continued_fraction_convergent_denominator(coefficients, m.suc)),
            Rat.from_nat(continued_fraction_convergent_denominator(coefficients, m.suc) *
                continued_fraction_convergent_denominator(coefficients, m.suc.suc)))
        Real.from_rat(Rat.from_nat(continued_fraction_convergent_denominator(
                coefficients, m.suc) *
                continued_fraction_convergent_denominator(coefficients, m.suc))) < Real.from_rat(Rat.from_nat(continued_fraction_convergent_denominator(
                coefficients, m.suc) *
                continued_fraction_convergent_denominator(coefficients, m.suc.suc)))
        continued_fraction_convergent_denominator_positive(coefficients, m.suc)
        continued_fraction_convergent_denominator_positive(coefficients, m.suc)
        real_from_nat_product_positive(
            continued_fraction_convergent_denominator(coefficients, m.suc),
            continued_fraction_convergent_denominator(coefficients, m.suc))
        Real.from_rat(Rat.from_nat(continued_fraction_convergent_denominator(
                coefficients, m.suc) *
                continued_fraction_convergent_denominator(coefficients, m.suc))) .is_positive
        continued_fraction_convergent_denominator_positive(coefficients, m.suc.suc)
        Nat.0 < continued_fraction_convergent_denominator(coefficients, m.suc.suc)
        real_from_nat_product_positive(
            continued_fraction_convergent_denominator(coefficients, m.suc),
            continued_fraction_convergent_denominator(coefficients, m.suc.suc))
        Real.from_rat(Rat.from_nat(continued_fraction_convergent_denominator(
                coefficients, m.suc) *
                continued_fraction_convergent_denominator(coefficients, m.suc.suc))) .is_positive
        pos_gt_zero(Real.from_rat(Rat.from_nat(continued_fraction_convergent_denominator(
                coefficients, m.suc) *
                continued_fraction_convergent_denominator(coefficients, m.suc))))
        Real.0 < Real.from_rat(Rat.from_nat(continued_fraction_convergent_denominator(
                coefficients, m.suc) *
                continued_fraction_convergent_denominator(coefficients, m.suc)))
        pos_gt_zero(Real.from_rat(Rat.from_nat(continued_fraction_convergent_denominator(
                coefficients, m.suc) *
                continued_fraction_convergent_denominator(coefficients, m.suc.suc))))
        Real.0 < Real.from_rat(Rat.from_nat(continued_fraction_convergent_denominator(
                coefficients, m.suc) *
                continued_fraction_convergent_denominator(coefficients, m.suc.suc)))
        inverse_on_positive_flips_inequality(
            Real.from_rat(Rat.from_nat(continued_fraction_convergent_denominator(
                coefficients, m.suc) *
                continued_fraction_convergent_denominator(coefficients, m.suc))),
            Real.from_rat(Rat.from_nat(continued_fraction_convergent_denominator(
                coefficients, m.suc) *
                continued_fraction_convergent_denominator(coefficients, m.suc.suc))))
        Real.from_rat(Rat.from_nat(continued_fraction_convergent_denominator(
                coefficients, m.suc) *
                continued_fraction_convergent_denominator(coefficients, m.suc.suc))) .inverse < Real.from_rat(Rat.from_nat(continued_fraction_convergent_denominator(
                coefficients, m.suc) *
                continued_fraction_convergent_denominator(coefficients, m.suc))) .inverse
        rat_from_nat_positive_ne_zero(
            continued_fraction_convergent_denominator(coefficients, m.suc) *
            continued_fraction_convergent_denominator(coefficients, m.suc.suc))
        Rat.from_nat(continued_fraction_convergent_denominator(coefficients, m.suc) *
            continued_fraction_convergent_denominator(coefficients, m.suc.suc)) != Rat.0
        real_from_nat_recip_inverse(
            continued_fraction_convergent_denominator(coefficients, m.suc) *
            continued_fraction_convergent_denominator(coefficients, m.suc.suc))
        Real.from_rat(Rat.1 / Rat.from_nat(continued_fraction_convergent_denominator(
                coefficients, m.suc) *
                continued_fraction_convergent_denominator(coefficients, m.suc.suc))) =
            Real.from_rat(Rat.from_nat(continued_fraction_convergent_denominator(
                coefficients, m.suc) *
                continued_fraction_convergent_denominator(coefficients, m.suc.suc))) .inverse
        Real.from_rat(Rat.1 / Rat.from_nat(continued_fraction_convergent_denominator(
                coefficients, m.suc) *
                continued_fraction_convergent_denominator(coefficients, m.suc.suc))) =
            continued_fraction_real_gap_bound(coefficients, m.suc)
        rat_from_nat_positive_ne_zero(
            continued_fraction_convergent_denominator(coefficients, m.suc) *
            continued_fraction_convergent_denominator(coefficients, m.suc))
        Rat.from_nat(continued_fraction_convergent_denominator(coefficients, m.suc) *
            continued_fraction_convergent_denominator(coefficients, m.suc)) != Rat.0
        real_from_nat_recip_inverse(
            continued_fraction_convergent_denominator(coefficients, m.suc) *
            continued_fraction_convergent_denominator(coefficients, m.suc))
        Real.from_rat(Rat.1 / Rat.from_nat(continued_fraction_convergent_denominator(
                coefficients, m.suc) *
                continued_fraction_convergent_denominator(coefficients, m.suc))) =
            Real.from_rat(Rat.from_nat(continued_fraction_convergent_denominator(
                coefficients, m.suc) *
                continued_fraction_convergent_denominator(coefficients, m.suc))) .inverse
        continued_fraction_real_gap_bound(coefficients, m.suc) < Real.from_rat(Rat.1 / Rat.from_nat(
                continued_fraction_convergent_denominator(coefficients, m.suc) *
                continued_fraction_convergent_denominator(coefficients, m.suc)))
    }
}

/// The real gap bound at the zeroth index is at most the reciprocal of the
/// squared initial denominator.
theorem continued_fraction_real_gap_zero_le_square(coefficients: Nat -> Nat) {
    positive_continued_fraction_sequence_tail(coefficients) implies continued_fraction_real_gap_bound(coefficients, Nat.0) <= Real.from_rat(Rat.1 / Rat.from_nat(
            continued_fraction_convergent_denominator(coefficients, Nat.0) *
            continued_fraction_convergent_denominator(coefficients, Nat.0)))
} by {
    if positive_continued_fraction_sequence_tail(coefficients) {
        continued_fraction_convergent_denominator_zero(coefficients)
        continued_fraction_convergent_denominator(coefficients, Nat.0) = Nat.1
        continued_fraction_convergent_denominator_ge_one(coefficients, Nat.1)
        Nat.1 <= continued_fraction_convergent_denominator(coefficients, Nat.1)
        nat_lte_imp_rat_lte(Nat.1,
            continued_fraction_convergent_denominator(coefficients, Nat.1))
        Rat.from_nat(Nat.1) <= Rat.from_nat(
            continued_fraction_convergent_denominator(coefficients, Nat.1))
        from_nat_one[Rat]
        Rat.from_nat(Nat.1) = Rat.1
        Rat.1 <= Rat.from_nat(continued_fraction_convergent_denominator(
            coefficients, Nat.1))
        continued_fraction_convergent_denominator_positive(coefficients, Nat.1)
        Nat.0 < continued_fraction_convergent_denominator(coefficients, Nat.1)
        nat_lt_imp_rat_lt(Nat.0, continued_fraction_convergent_denominator(
            coefficients, Nat.1))
        Rat.from_nat(Nat.0) < Rat.from_nat(continued_fraction_convergent_denominator(
            coefficients, Nat.1))
        from_nat_zero[Rat]
        Rat.from_nat(Nat.0) = Rat.0
        Rat.0 < Rat.from_nat(continued_fraction_convergent_denominator(
            coefficients, Nat.1))
        zero_lt_imp_pos(Rat.from_nat(continued_fraction_convergent_denominator(
            coefficients, Nat.1)))
        Rat.from_nat(continued_fraction_convergent_denominator(
            coefficients, Nat.1)).is_positive
        Rat.1.is_positive
        Rat.1 * Rat.1 = Rat.1
        Rat.1 * Rat.from_nat(continued_fraction_convergent_denominator(
            coefficients, Nat.1)) = Rat.from_nat(
            continued_fraction_convergent_denominator(coefficients, Nat.1))
        Rat.1 <= Rat.1 * Rat.from_nat(continued_fraction_convergent_denominator(
            coefficients, Nat.1))
        cross_mul_lte(
            Rat.1,
            Rat.from_nat(continued_fraction_convergent_denominator(
                coefficients, Nat.1)),
            Rat.1, Rat.1)
        Rat.1 / Rat.from_nat(continued_fraction_convergent_denominator(
                coefficients, Nat.1)) <= Rat.1 / Rat.1
        Rat.1 / Rat.1 = Rat.1
        Rat.1 / Rat.from_nat(continued_fraction_convergent_denominator(
                coefficients, Nat.1)) <= Rat.1
        mul_one_left(continued_fraction_convergent_denominator(coefficients, Nat.0))
        continued_fraction_convergent_denominator(coefficients, Nat.0) *
            continued_fraction_convergent_denominator(coefficients, Nat.1) =
            continued_fraction_convergent_denominator(coefficients, Nat.1)
        continued_fraction_gap_bound(coefficients, Nat.0) =
            Rat.1 / Rat.from_nat(continued_fraction_convergent_denominator(
                coefficients, Nat.0) *
                continued_fraction_convergent_denominator(coefficients, Nat.1))
        continued_fraction_gap_bound(coefficients, Nat.0) =
            Rat.1 / Rat.from_nat(continued_fraction_convergent_denominator(
                coefficients, Nat.1))
        continued_fraction_real_gap_bound(coefficients, Nat.0) =
            Real.from_rat(continued_fraction_gap_bound(coefficients, Nat.0))
        continued_fraction_real_gap_bound(coefficients, Nat.0) =
            Real.from_rat(Rat.1 / Rat.from_nat(
                continued_fraction_convergent_denominator(coefficients, Nat.1)))
        mul_one_right(continued_fraction_convergent_denominator(coefficients, Nat.0))
        continued_fraction_convergent_denominator(coefficients, Nat.0) *
            continued_fraction_convergent_denominator(coefficients, Nat.0) =
            continued_fraction_convergent_denominator(coefficients, Nat.0)
        continued_fraction_convergent_denominator(coefficients, Nat.0) = Nat.1
        continued_fraction_convergent_denominator(coefficients, Nat.0) *
            continued_fraction_convergent_denominator(coefficients, Nat.0) = Nat.1
        Rat.1 / Rat.from_nat(
                continued_fraction_convergent_denominator(coefficients, Nat.0) *
                continued_fraction_convergent_denominator(coefficients, Nat.0)) =
            Rat.1 / Rat.from_nat(Nat.1)
        from_nat_one[Rat]
        Rat.from_nat(Nat.1) = Rat.1
        Rat.1 / Rat.from_nat(Nat.1) = Rat.1
        Rat.1 / Rat.from_nat(
                continued_fraction_convergent_denominator(coefficients, Nat.0) *
                continued_fraction_convergent_denominator(coefficients, Nat.0)) =
            Rat.1
        from_rat_maintains_lte(
            Rat.1 / Rat.from_nat(continued_fraction_convergent_denominator(
                coefficients, Nat.1)),
            Rat.1)
        Real.from_rat(Rat.1 / Rat.from_nat(
                continued_fraction_convergent_denominator(coefficients, Nat.1))) <= Real.from_rat(Rat.1)
        continued_fraction_real_gap_bound(coefficients, Nat.0) <= Real.from_rat(Rat.1 / Rat.from_nat(
                continued_fraction_convergent_denominator(coefficients, Nat.0) *
                continued_fraction_convergent_denominator(coefficients, Nat.0)))
    }
}

/// Negating a real difference interchanges the operands.
theorem real_neg_sub(a: Real, b: Real) {
    -(a - b) = b - a
} by {
}

/// Negation reverses strict real inequalities.
theorem real_lt_neg_flip(a: Real, b: Real) {
    a < b implies -b < -a
} by {
}

/// Negating a real number does not change its absolute value.
theorem real_neg_abs(a: Real) {
    (-a).abs = a.abs
} by {
}

/// A real strictly between two reals is strictly closer to the right one than
/// the left one is.
theorem real_strict_between_abs_lt_right(a: Real, e: Real, b: Real) {
    a < e and e < b implies (b - e).abs < (b - a).abs
} by {
    if a < e and e < b {
        lt_trans(a, e, b)
        a < b
        lt_imp_minus_pos(e, b)
        (b - e).is_positive
        pos_imp_eq_abs(b - e)
        (b - e).abs = b - e
        lt_imp_minus_pos(a, b)
        (b - a).is_positive
        pos_imp_eq_abs(b - a)
        (b - a).abs = b - a
        lt_add_right(a, e, -b)
        a + -b < e + -b
        a - b < e - b
        -(e - b) < -(a - b)
        b - e < b - a
        (b - e).abs < (b - a).abs
    }
}

/// The classical approximation estimate: the real limit of the convergents is
/// within `1 / q_n^2` of the `n`-th convergent.
theorem continued_fraction_approximation_estimate(
    coefficients: Nat -> Nat, n: Nat
) {
    positive_continued_fraction_sequence_tail(coefficients) implies (continued_fraction_real_limit(coefficients) - continued_fraction_real_convergent_value(coefficients, n)).abs < Real.from_rat(Rat.1 / Rat.from_nat(
            continued_fraction_convergent_denominator(coefficients, n) *
            continued_fraction_convergent_denominator(coefficients, n)))
} by {
    if positive_continued_fraction_sequence_tail(coefficients) {
        nat_even_or_odd(n)
        if exists(k: Nat) { n = Nat.2 * k } {
            let k: Nat satisfy {
                n = Nat.2 * k
            }
            n = Nat.2 * k
            continued_fraction_real_even_lt_limit(coefficients, k)
            continued_fraction_real_even_convergent(coefficients, k) < continued_fraction_real_limit(coefficients)
            continued_fraction_real_limit_lt_odd(coefficients, k)
            continued_fraction_real_limit(coefficients) < continued_fraction_real_odd_convergent(coefficients, k)
            real_strict_between_abs_lt(
                continued_fraction_real_even_convergent(coefficients, k),
                continued_fraction_real_limit(coefficients),
                continued_fraction_real_odd_convergent(coefficients, k))
            (continued_fraction_real_limit(coefficients) -
                continued_fraction_real_even_convergent(coefficients, k)).abs < (continued_fraction_real_odd_convergent(coefficients, k) - continued_fraction_real_even_convergent(coefficients, k)).abs
            continued_fraction_real_convergent_value(coefficients, n) =
                continued_fraction_real_even_convergent(coefficients, k)
            (continued_fraction_real_limit(coefficients) -
                continued_fraction_real_convergent_value(coefficients, n)).abs =
                (continued_fraction_real_limit(coefficients) -
                    continued_fraction_real_even_convergent(coefficients, k)).abs
            continued_fraction_real_convergent_value(coefficients, n.suc) =
                continued_fraction_real_odd_convergent(coefficients, k)
            (continued_fraction_real_odd_convergent(coefficients, k) -
                continued_fraction_real_even_convergent(coefficients, k)).abs =
                (continued_fraction_real_convergent_value(coefficients, n.suc) -
                    continued_fraction_real_convergent_value(coefficients, n)).abs
            (continued_fraction_real_limit(coefficients) -
                continued_fraction_real_convergent_value(coefficients, n)).abs < (continued_fraction_real_convergent_value(coefficients, n.suc) - continued_fraction_real_convergent_value(coefficients, n)).abs
            continued_fraction_real_gap_eq_embedded_gap(coefficients, n)
            (continued_fraction_real_convergent_value(coefficients, n.suc) -
                continued_fraction_real_convergent_value(coefficients, n)).abs =
                continued_fraction_real_gap_bound(coefficients, n)
            (continued_fraction_real_limit(coefficients) -
                continued_fraction_real_convergent_value(coefficients, n)).abs < continued_fraction_real_gap_bound(coefficients, n)
            if n = Nat.0 {
                continued_fraction_real_gap_zero_le_square(coefficients)
                continued_fraction_real_gap_bound(coefficients, Nat.0) <= Real.from_rat(Rat.1 / Rat.from_nat(
                        continued_fraction_convergent_denominator(coefficients, Nat.0) *
                        continued_fraction_convergent_denominator(coefficients, Nat.0)))
                continued_fraction_real_gap_bound(coefficients, n) <= Real.from_rat(Rat.1 / Rat.from_nat(
                        continued_fraction_convergent_denominator(coefficients, n) *
                        continued_fraction_convergent_denominator(coefficients, n)))
                lt_of_lt_of_lte(
                    (continued_fraction_real_limit(coefficients) -
                        continued_fraction_real_convergent_value(coefficients, n)).abs,
                    continued_fraction_real_gap_bound(coefficients, n),
                    Real.from_rat(Rat.1 / Rat.from_nat(
                        continued_fraction_convergent_denominator(coefficients, n) *
                        continued_fraction_convergent_denominator(coefficients, n))))
                (continued_fraction_real_limit(coefficients) -
                    continued_fraction_real_convergent_value(coefficients, n)).abs < Real.from_rat(Rat.1 / Rat.from_nat(
                        continued_fraction_convergent_denominator(coefficients, n) *
                        continued_fraction_convergent_denominator(coefficients, n)))
                n = Nat.0 implies (continued_fraction_real_limit(coefficients) - continued_fraction_real_convergent_value(coefficients, n)).abs < Real.from_rat(Rat.1 / Rat.from_nat(
                        continued_fraction_convergent_denominator(coefficients, n) *
                        continued_fraction_convergent_denominator(coefficients, n)))
            }
            if n != Nat.0 {
                zero_or_suc(n)
                n = Nat.0 or exists(m: Nat) { m.suc = n }
                if n = Nat.0 {
                    false
                }
                if exists(m: Nat) { m.suc = n } {
                    let m: Nat satisfy {
                        m.suc = n
                    }
                    continued_fraction_real_gap_lt_square(coefficients, m)
                    continued_fraction_real_gap_bound(coefficients, m.suc) < Real.from_rat(Rat.1 / Rat.from_nat(
                            continued_fraction_convergent_denominator(coefficients, m.suc) *
                            continued_fraction_convergent_denominator(coefficients, m.suc)))
                    continued_fraction_real_gap_bound(coefficients, n) < Real.from_rat(Rat.1 / Rat.from_nat(
                            continued_fraction_convergent_denominator(coefficients, n) *
                            continued_fraction_convergent_denominator(coefficients, n)))
                    lt_trans(
                        (continued_fraction_real_limit(coefficients) -
                            continued_fraction_real_convergent_value(coefficients, n)).abs,
                        continued_fraction_real_gap_bound(coefficients, n),
                        Real.from_rat(Rat.1 / Rat.from_nat(
                            continued_fraction_convergent_denominator(coefficients, n) *
                            continued_fraction_convergent_denominator(coefficients, n))))
                    (continued_fraction_real_limit(coefficients) -
                        continued_fraction_real_convergent_value(coefficients, n)).abs < Real.from_rat(Rat.1 / Rat.from_nat(
                            continued_fraction_convergent_denominator(coefficients, n) *
                            continued_fraction_convergent_denominator(coefficients, n)))
                }
                (continued_fraction_real_limit(coefficients) -
                    continued_fraction_real_convergent_value(coefficients, n)).abs < Real.from_rat(Rat.1 / Rat.from_nat(
                        continued_fraction_convergent_denominator(coefficients, n) *
                        continued_fraction_convergent_denominator(coefficients, n)))
            }
            (continued_fraction_real_limit(coefficients) -
                continued_fraction_real_convergent_value(coefficients, n)).abs < Real.from_rat(Rat.1 / Rat.from_nat(
                    continued_fraction_convergent_denominator(coefficients, n) *
                    continued_fraction_convergent_denominator(coefficients, n)))
        }
        if exists(k: Nat) { n = (Nat.2 * k).suc } {
            let k: Nat satisfy {
                n = (Nat.2 * k).suc
            }
            n = (Nat.2 * k).suc
            continued_fraction_real_even_lt_limit(coefficients, k.suc)
            continued_fraction_real_even_convergent(coefficients, k.suc) < continued_fraction_real_limit(coefficients)
            continued_fraction_real_limit_lt_odd(coefficients, k)
            continued_fraction_real_limit(coefficients) < continued_fraction_real_odd_convergent(coefficients, k)
            real_strict_between_abs_lt_right(
                continued_fraction_real_even_convergent(coefficients, k.suc),
                continued_fraction_real_limit(coefficients),
                continued_fraction_real_odd_convergent(coefficients, k))
            (continued_fraction_real_odd_convergent(coefficients, k) -
                continued_fraction_real_limit(coefficients)).abs < (continued_fraction_real_odd_convergent(coefficients, k) - continued_fraction_real_even_convergent(coefficients, k.suc)).abs
            continued_fraction_real_convergent_value(coefficients, n.suc) =
                continued_fraction_real_even_convergent(coefficients, k.suc)
            continued_fraction_real_convergent_value(coefficients, n) =
                continued_fraction_real_odd_convergent(coefficients, k)
            (continued_fraction_real_limit(coefficients) -
                continued_fraction_real_convergent_value(coefficients, n)).abs =
                (continued_fraction_real_limit(coefficients) -
                    continued_fraction_real_odd_convergent(coefficients, k)).abs
            real_neg_sub(
                continued_fraction_real_odd_convergent(coefficients, k),
                continued_fraction_real_limit(coefficients))
            -(continued_fraction_real_odd_convergent(coefficients, k) -
                continued_fraction_real_limit(coefficients)) =
                continued_fraction_real_limit(coefficients) -
                continued_fraction_real_odd_convergent(coefficients, k)
            real_neg_abs(continued_fraction_real_odd_convergent(coefficients, k) -
                continued_fraction_real_limit(coefficients))
            (-(continued_fraction_real_odd_convergent(coefficients, k) -
                continued_fraction_real_limit(coefficients))).abs =
                (continued_fraction_real_odd_convergent(coefficients, k) -
                    continued_fraction_real_limit(coefficients)).abs
            (continued_fraction_real_limit(coefficients) -
                continued_fraction_real_odd_convergent(coefficients, k)).abs =
                (continued_fraction_real_odd_convergent(coefficients, k) -
                    continued_fraction_real_limit(coefficients)).abs
            (continued_fraction_real_limit(coefficients) -
                continued_fraction_real_convergent_value(coefficients, n)).abs =
                (continued_fraction_real_odd_convergent(coefficients, k) -
                    continued_fraction_real_limit(coefficients)).abs
            (continued_fraction_real_limit(coefficients) -
                continued_fraction_real_convergent_value(coefficients, n)).abs < (continued_fraction_real_odd_convergent(coefficients, k) - continued_fraction_real_even_convergent(coefficients, k.suc)).abs
            (continued_fraction_real_odd_convergent(coefficients, k) -
                continued_fraction_real_even_convergent(coefficients, k.suc)).abs =
                (continued_fraction_real_convergent_value(coefficients, n) -
                    continued_fraction_real_convergent_value(coefficients, n.suc)).abs
            (continued_fraction_real_limit(coefficients) -
                continued_fraction_real_convergent_value(coefficients, n)).abs < (continued_fraction_real_convergent_value(coefficients, n) - continued_fraction_real_convergent_value(coefficients, n.suc)).abs
            real_neg_sub(
                continued_fraction_real_convergent_value(coefficients, n),
                continued_fraction_real_convergent_value(coefficients, n.suc))
            -(continued_fraction_real_convergent_value(coefficients, n) -
                continued_fraction_real_convergent_value(coefficients, n.suc)) =
                continued_fraction_real_convergent_value(coefficients, n.suc) -
                continued_fraction_real_convergent_value(coefficients, n)
            real_neg_abs(continued_fraction_real_convergent_value(coefficients, n) -
                continued_fraction_real_convergent_value(coefficients, n.suc))
            (-(continued_fraction_real_convergent_value(coefficients, n) -
                continued_fraction_real_convergent_value(coefficients, n.suc))).abs =
                (continued_fraction_real_convergent_value(coefficients, n) -
                    continued_fraction_real_convergent_value(coefficients, n.suc)).abs
            (continued_fraction_real_convergent_value(coefficients, n) -
                continued_fraction_real_convergent_value(coefficients, n.suc)).abs =
                (continued_fraction_real_convergent_value(coefficients, n.suc) -
                    continued_fraction_real_convergent_value(coefficients, n)).abs
            (continued_fraction_real_limit(coefficients) -
                continued_fraction_real_convergent_value(coefficients, n)).abs < (continued_fraction_real_convergent_value(coefficients, n.suc) - continued_fraction_real_convergent_value(coefficients, n)).abs
            continued_fraction_real_gap_eq_embedded_gap(coefficients, n)
            (continued_fraction_real_convergent_value(coefficients, n.suc) -
                continued_fraction_real_convergent_value(coefficients, n)).abs =
                continued_fraction_real_gap_bound(coefficients, n)
            (continued_fraction_real_limit(coefficients) -
                continued_fraction_real_convergent_value(coefficients, n)).abs < continued_fraction_real_gap_bound(coefficients, n)
            continued_fraction_real_gap_lt_square(coefficients, Nat.2 * k)
            continued_fraction_real_gap_bound(coefficients, (Nat.2 * k).suc) < Real.from_rat(Rat.1 / Rat.from_nat(
                    continued_fraction_convergent_denominator(coefficients, (Nat.2 * k).suc) *
                    continued_fraction_convergent_denominator(coefficients, (Nat.2 * k).suc)))
            continued_fraction_real_gap_bound(coefficients, n) < Real.from_rat(Rat.1 / Rat.from_nat(
                    continued_fraction_convergent_denominator(coefficients, n) *
                    continued_fraction_convergent_denominator(coefficients, n)))
            lt_trans(
                (continued_fraction_real_limit(coefficients) -
                    continued_fraction_real_convergent_value(coefficients, n)).abs,
                continued_fraction_real_gap_bound(coefficients, n),
                Real.from_rat(Rat.1 / Rat.from_nat(
                    continued_fraction_convergent_denominator(coefficients, n) *
                    continued_fraction_convergent_denominator(coefficients, n))))
            (continued_fraction_real_limit(coefficients) -
                continued_fraction_real_convergent_value(coefficients, n)).abs < Real.from_rat(Rat.1 / Rat.from_nat(
                    continued_fraction_convergent_denominator(coefficients, n) *
                    continued_fraction_convergent_denominator(coefficients, n)))
        }
        (continued_fraction_real_limit(coefficients) -
            continued_fraction_real_convergent_value(coefficients, n)).abs < Real.from_rat(Rat.1 / Rat.from_nat(
                continued_fraction_convergent_denominator(coefficients, n) *
                continued_fraction_convergent_denominator(coefficients, n)))
    }
}

/// A positive real embedding forces the rational to be positive.
theorem rat_positive_of_real_positive(r: Rat) {
    Real.0 < Real.from_rat(r) implies r.is_positive
} by {
    if Real.0 < Real.from_rat(r) {
        rat_total(r, Rat.0)
        r <= Rat.0 or Rat.0 <= r
        if r <= Rat.0 {
            from_rat_maintains_lte(r, Rat.0)
            Real.from_rat(r) <= Real.from_rat(Rat.0)
            Real.from_rat(Rat.0) = Real.0
            Real.from_rat(r) <= Real.0
            false
        }
        if Rat.0 <= r {
            if r = Rat.0 {
                Real.from_rat(r) = Real.from_rat(Rat.0)
                Real.from_rat(Rat.0) = Real.0
                Real.from_rat(r) = Real.0
                false
            }
            Rat.0 < r
            zero_lt_imp_pos(r)
            r.is_positive
        }
        r.is_positive
    }
}

/// A nonnegative real equals its absolute value.
theorem real_abs_of_nonneg(a: Real) {
    Real.0 <= a implies a.abs = a
} by {
    if Real.0 <= a {
        if a = Real.0 {
            a.abs = a
        }
        if a != Real.0 {
            Real.0 < a
            gt_zero_imp_pos(a)
            a.is_positive
            pos_imp_eq_abs(a)
            a.abs = a
        }
        a.abs = a
    }
}

/// Doubling a natural inequality can be cancelled.
theorem lte_double_imp_lte(m: Nat, n: Nat) {
    Nat.2 * n <= Nat.2 * m implies n <= m
} by {
    if Nat.2 * n <= Nat.2 * m {
        lt_or_lte(m, n)
        m < n or n <= m
        if m < n {
            lt_mul_both(Nat.2, m, n)
            Nat.2 * m < Nat.2 * n
            false
        }
        if n <= m {
            n <= m
        }
        n <= m
    }
}

/// The rational gap bound at an even index is at most the unit fraction with
/// the doubled index.
theorem continued_fraction_gap_bound_le_recip_suc(coefficients: Nat -> Nat, k: Nat) {
    positive_continued_fraction_sequence_tail(coefficients) implies continued_fraction_gap_bound(coefficients, Nat.2 * k) <= iop(k)
} by {
    if positive_continued_fraction_sequence_tail(coefficients) {
        continued_fraction_convergent_denominator_ge_one(coefficients, Nat.2 * k)
        Nat.1 <= continued_fraction_convergent_denominator(coefficients, Nat.2 * k)
        continued_fraction_convergent_denominator_ge_index(coefficients, (Nat.2 * k).suc)
        (Nat.2 * k).suc <= continued_fraction_convergent_denominator(
            coefficients, (Nat.2 * k).suc)
        lte_mul_both(
            continued_fraction_convergent_denominator(coefficients, Nat.2 * k),
            (Nat.2 * k).suc,
            continued_fraction_convergent_denominator(coefficients, (Nat.2 * k).suc))
        continued_fraction_convergent_denominator(coefficients, Nat.2 * k) *
                (Nat.2 * k).suc <= continued_fraction_convergent_denominator(coefficients, Nat.2 * k) * continued_fraction_convergent_denominator(coefficients, (Nat.2 * k).suc)
        lte_mul_both(
            (Nat.2 * k).suc,
            Nat.1,
            continued_fraction_convergent_denominator(coefficients, Nat.2 * k))
        (Nat.2 * k).suc * Nat.1 <= (Nat.2 * k).suc * continued_fraction_convergent_denominator(coefficients, Nat.2 * k)
        mul_one_right((Nat.2 * k).suc)
        (Nat.2 * k).suc * Nat.1 = (Nat.2 * k).suc
        (Nat.2 * k).suc <= (Nat.2 * k).suc * continued_fraction_convergent_denominator(coefficients, Nat.2 * k)
        mul_comm(
            (Nat.2 * k).suc,
            continued_fraction_convergent_denominator(coefficients, Nat.2 * k))
        (Nat.2 * k).suc *
                continued_fraction_convergent_denominator(coefficients, Nat.2 * k) =
            continued_fraction_convergent_denominator(coefficients, Nat.2 * k) *
                (Nat.2 * k).suc
        (Nat.2 * k).suc <= continued_fraction_convergent_denominator(coefficients, Nat.2 * k) * (Nat.2 * k).suc
        lte_trans(
            (Nat.2 * k).suc,
            continued_fraction_convergent_denominator(coefficients, Nat.2 * k) *
                (Nat.2 * k).suc,
            continued_fraction_convergent_denominator(coefficients, Nat.2 * k) *
                continued_fraction_convergent_denominator(coefficients, (Nat.2 * k).suc))
        (Nat.2 * k).suc <= continued_fraction_convergent_denominator(coefficients, Nat.2 * k) * continued_fraction_convergent_denominator(coefficients, (Nat.2 * k).suc)
        Nat.0 <= k
        lte_add_right(Nat.0, k, k)
        Nat.0 + k <= k + k
        add_zero_left(k)
        Nat.0 + k = k
        k <= k + k
        mul_two_left(k)
        k + k = Nat.2 * k
        k <= Nat.2 * k
        lte_add_right(k, Nat.2 * k, Nat.1)
        k + Nat.1 <= Nat.2 * k + Nat.1
        k.suc <= (Nat.2 * k).suc
        lte_trans(
            k.suc,
            (Nat.2 * k).suc,
            continued_fraction_convergent_denominator(coefficients, Nat.2 * k) *
                continued_fraction_convergent_denominator(coefficients, (Nat.2 * k).suc))
        k.suc <= continued_fraction_convergent_denominator(coefficients, Nat.2 * k) * continued_fraction_convergent_denominator(coefficients, (Nat.2 * k).suc)
        nat_lte_imp_rat_lte(
            k.suc,
            continued_fraction_convergent_denominator(coefficients, Nat.2 * k) *
                continued_fraction_convergent_denominator(coefficients, (Nat.2 * k).suc))
        Rat.from_nat(k.suc) <= Rat.from_nat(
            continued_fraction_convergent_denominator(coefficients, Nat.2 * k) *
                continued_fraction_convergent_denominator(coefficients, (Nat.2 * k).suc))
        continued_fraction_convergent_denominator_positive(coefficients, Nat.2 * k)
        Nat.0 < continued_fraction_convergent_denominator(coefficients, Nat.2 * k)
        continued_fraction_convergent_denominator_positive(coefficients, (Nat.2 * k).suc)
        Nat.0 < continued_fraction_convergent_denominator(coefficients, (Nat.2 * k).suc)
        nat_mul_positive(
            continued_fraction_convergent_denominator(coefficients, Nat.2 * k),
            continued_fraction_convergent_denominator(coefficients, (Nat.2 * k).suc))
        Nat.0 < continued_fraction_convergent_denominator(coefficients, Nat.2 * k) and
            Nat.0 < continued_fraction_convergent_denominator(coefficients, (Nat.2 * k).suc)
        Nat.0 < continued_fraction_convergent_denominator(coefficients, Nat.2 * k) *
            continued_fraction_convergent_denominator(coefficients, (Nat.2 * k).suc)
        rat_from_nat_positive_ne_zero(
            continued_fraction_convergent_denominator(coefficients, Nat.2 * k) *
            continued_fraction_convergent_denominator(coefficients, (Nat.2 * k).suc))
        Rat.from_nat(continued_fraction_convergent_denominator(coefficients, Nat.2 * k) *
            continued_fraction_convergent_denominator(coefficients, (Nat.2 * k).suc)) .is_positive
        lt_suc(k)
        Nat.0 < k.suc
        rat_from_nat_positive_ne_zero(k.suc)
        Rat.from_nat(k.suc) != Rat.0
        nat_lt_imp_rat_lt(Nat.0, k.suc)
        Rat.from_nat(Nat.0) < Rat.from_nat(k.suc)
        from_nat_zero[Rat]
        Rat.from_nat(Nat.0) = Rat.0
        Rat.0 < Rat.from_nat(k.suc)
        zero_lt_imp_pos(Rat.from_nat(k.suc))
        Rat.from_nat(k.suc).is_positive
        Rat.1 * Rat.from_nat(k.suc) = Rat.from_nat(k.suc)
        Rat.1 * Rat.from_nat(
                continued_fraction_convergent_denominator(coefficients, Nat.2 * k) *
                continued_fraction_convergent_denominator(coefficients, (Nat.2 * k).suc)) =
            Rat.from_nat(
                continued_fraction_convergent_denominator(coefficients, Nat.2 * k) *
                continued_fraction_convergent_denominator(coefficients, (Nat.2 * k).suc))
        Rat.1 * Rat.from_nat(k.suc) <= Rat.1 * Rat.from_nat(
            continued_fraction_convergent_denominator(coefficients, Nat.2 * k) *
            continued_fraction_convergent_denominator(coefficients, (Nat.2 * k).suc))
        cross_mul_lte(
            Rat.1,
            Rat.from_nat(
                continued_fraction_convergent_denominator(coefficients, Nat.2 * k) *
                continued_fraction_convergent_denominator(coefficients, (Nat.2 * k).suc)),
            Rat.1,
            Rat.from_nat(k.suc))
        Rat.1 / Rat.from_nat(
                continued_fraction_convergent_denominator(coefficients, Nat.2 * k) *
                continued_fraction_convergent_denominator(coefficients, (Nat.2 * k).suc)) <= Rat.1 / Rat.from_nat(k.suc)
        continued_fraction_gap_bound(coefficients, Nat.2 * k) =
            Rat.1 / Rat.from_nat(
                continued_fraction_convergent_denominator(coefficients, Nat.2 * k) *
                continued_fraction_convergent_denominator(coefficients, (Nat.2 * k).suc))
        continued_fraction_gap_bound(coefficients, Nat.2 * k) <= Rat.1 / Rat.from_nat(k.suc)
        from_nat_add(k, Nat.1)
        Rat.from_nat(k + Nat.1) = Rat.from_nat(k) + Rat.from_nat(Nat.1)
        from_nat_one[Rat]
        Rat.from_nat(Nat.1) = Rat.1
        Rat.from_nat(k + Nat.1) = Rat.from_nat(k) + Rat.1
        Rat.from_nat(k + Nat.1) = Rat.1 + Rat.from_nat(k)
        k + Nat.1 = k.suc
        Rat.from_nat(k.suc) = Rat.1 + Rat.from_nat(k)
        Rat.1 / Rat.from_nat(k.suc) = Rat.1 / (Rat.1 + Rat.from_nat(k))
        iop(k) = Rat.1 / (Rat.1 + Rat.from_nat(k))
        Rat.1 / Rat.from_nat(k.suc) = iop(k)
        continued_fraction_gap_bound(coefficients, Nat.2 * k) <= iop(k)
    }
}

/// The real gap bound at an even index is at most the embedded unit fraction.
theorem continued_fraction_real_gap_even_le_recip_suc(
    coefficients: Nat -> Nat, k: Nat
) {
    positive_continued_fraction_sequence_tail(coefficients) implies continued_fraction_real_gap_bound(coefficients, Nat.2 * k) <= Real.from_rat(iop(k))
} by {
    if positive_continued_fraction_sequence_tail(coefficients) {
        continued_fraction_gap_bound_le_recip_suc(coefficients, k)
        continued_fraction_gap_bound(coefficients, Nat.2 * k) <= iop(k)
        from_rat_maintains_lte(
            continued_fraction_gap_bound(coefficients, Nat.2 * k), iop(k))
        Real.from_rat(continued_fraction_gap_bound(coefficients, Nat.2 * k)) <= Real.from_rat(iop(k))
        continued_fraction_real_gap_bound(coefficients, Nat.2 * k) =
            Real.from_rat(continued_fraction_gap_bound(coefficients, Nat.2 * k))
        continued_fraction_real_gap_bound(coefficients, Nat.2 * k) <= Real.from_rat(iop(k))
    }
}

// /// The embedded unit fractions converge to zero.
// theorem continued_fraction_real_recip_suc_converges_to_zero {
//     converges_to(
//         function(k: Nat) { Real.from_rat(iop(k)) },
//         Real.0)
// } by {
//     forall(eps: Real) {
//         if eps.is_positive {
//             pos_gt_zero(eps)
//             Real.0 < eps
//             rat_between_reals(Real.0, eps)
//             exists(r: Rat) { Real.0 < Real.from_rat(r) and Real.from_rat(r) < eps }
//             let r: Rat satisfy {
//                 Real.0 < Real.from_rat(r) and Real.from_rat(r) < eps
//             }
//             rat_positive_of_real_positive(r)
//             r.is_positive
//             iop_gets_lt(r)
//             exists(n: Nat) {
//                 forall(i: Nat) {
//                     n <= i implies iop(i) < r
//                 }
//             }
//             let n: Nat satisfy {
//                 forall(i: Nat) {
//                     n <= i implies iop(i) < r
//                 }
//             }
//             forall(i: Nat) {
//                 if n <= i {
//                     iop(i) < r
//                     from_rat_maintains_lt(iop(i), r)
//                     Real.from_rat(iop(i)) < Real.from_rat(r)
//                     Real.from_rat(r) < eps
//                     lt_trans(Real.from_rat(iop(i)), Real.from_rat(r), eps)
//                     Real.from_rat(iop(i)) < eps
//                     iop_pos(i)
//                     iop(i).is_positive
//                     pos_imp_zero_lt(iop(i))
//                     Rat.0 < iop(i)
//                     from_rat_maintains_lt(Rat.0, iop(i))
//                     Real.from_rat(Rat.0) < Real.from_rat(iop(i))
//                     Real.from_rat(Rat.0) = Real.0
//                     Real.0 < Real.from_rat(iop(i))
//                     gt_zero_imp_pos(Real.from_rat(iop(i)))
//                     Real.from_rat(iop(i)).is_positive
//                     pos_gt_zero(Real.from_rat(iop(i)))
//                     Real.0 < Real.from_rat(iop(i))
//                     real_abs_of_nonneg(Real.from_rat(iop(i)))
//                     Real.from_rat(iop(i)).abs = Real.from_rat(iop(i))
//                     (Real.from_rat(iop(i)) - Real.0).abs < eps
//                     Real.from_rat(iop(i)).is_close(Real.0, eps)
//                 }
//             }
//             tail_bound(
//                 function(k: Nat) { Real.from_rat(iop(k)) },
//                 Real.0, n, eps)
//             exists(n1: Nat) {
//                 tail_bound(
//                     function(k: Nat) { Real.from_rat(iop(k)) },
//                     Real.0, n1, eps)
//             }
//         }
//     }
//     converges_to(
//         function(k: Nat) { Real.from_rat(iop(k)) },
//         Real.0)
// }

/// Real less-than-or-equal is transitive.
theorem real_lte_trans(a: Real, b: Real, c: Real) {
    a <= b and b <= c implies a <= c
} by {
    if a <= b and b <= c {
        a <= b = forall(r: Rat) { a.gt_rat(r) implies b.gt_rat(r) }
        b <= c = forall(r: Rat) { b.gt_rat(r) implies c.gt_rat(r) }
        forall(r: Rat) {
            if a.gt_rat(r) {
                forall(r1: Rat) { a.gt_rat(r1) implies b.gt_rat(r1) }
                a.gt_rat(r) implies b.gt_rat(r)
                b.gt_rat(r)
                forall(r2: Rat) { b.gt_rat(r2) implies c.gt_rat(r2) }
                b.gt_rat(r) implies c.gt_rat(r)
                c.gt_rat(r)
            }
        }
        a <= c
    }
}

/// A real bounded below a strict upper bound is strictly below it.
theorem real_lt_of_lte_of_lt(a: Real, b: Real, c: Real) {
    a <= b and b < c implies a < c
} by {
    if a <= b and b < c {
        lt_imp_lte(b, c)
        b <= c
        real_lte_trans(a, b, c)
        a <= c
        if a = c {
            c <= b
            lte_imp_not_lt(c, b)
            false
        }
        a < c
    }
}

// /// A nonnegative sequence squeezed by a vanishing sequence converges to zero.
// theorem cf_converges_to_squeeze_nonneg(a: Nat -> Real, b: Nat -> Real) {
//     (forall(n: Nat) { Real.0 <= a(n) and a(n) <= b(n) }) and
//         converges_to(b, Real.0) implies converges_to(a, Real.0)
// } by {
//     if forall(n: Nat) { Real.0 <= a(n) and a(n) <= b(n) } and
//             converges_to(b, Real.0) {
//         forall(eps: Real) {
//             if eps.is_positive {
//                 converges_to(b, Real.0) = forall(e: Real) {
//                     e.is_positive implies exists(n: Nat) {
//                         tail_bound(b, Real.0, n, e)
//                     }
//                 }
//                 exists(n: Nat) {
//                     tail_bound(b, Real.0, n, eps)
//                 }
//                 let n: Nat satisfy {
//                     tail_bound(b, Real.0, n, eps)
//                 }
//                 forall(i: Nat) {
//                     if n <= i {
//                         tail_bound_implies_is_close(b, Real.0, n, eps, i)
//                         b(i).is_close(Real.0, eps)
//                         (b(i) - Real.0).abs < eps
//                         b(i).abs < eps
//                         Real.0 <= a(i) and a(i) <= b(i)
//                         a(i) <= b(i)
//                         lte_abs(b(i))
//                         b(i) <= b(i).abs
//                         real_lte_trans(a(i), b(i), b(i).abs)
//                         a(i) <= b(i).abs
//                         real_lt_of_lte_of_lt(a(i), b(i).abs, eps)
//                         a(i) < eps
//                         Real.0 <= a(i)
//                         real_abs_of_nonneg(a(i))
//                         a(i).abs = a(i)
//                         (a(i) - Real.0).abs < eps
//                         a(i).is_close(Real.0, eps)
//                     }
//                 }
//                 tail_bound(a, Real.0, n, eps)
//                 exists(n1: Nat) {
//                     tail_bound(a, Real.0, n1, eps)
//                 }
//             }
//         }
//         converges_to(a, Real.0)
//     }
// }

// /// The real gap bounds at even indices converge to zero.
// theorem continued_fraction_real_gap_even_converges_to_zero(
//     coefficients: Nat -> Nat
// ) {
//     positive_continued_fraction_sequence_tail(coefficients) implies converges_to(
//             function(k: Nat) {
//                 continued_fraction_real_gap_bound(coefficients, Nat.2 * k)
//             },
//             Real.0)
// } by {
//     if positive_continued_fraction_sequence_tail(coefficients) {
//         forall(k: Nat) {
//             continued_fraction_real_gap_even_le_recip_suc(coefficients, k)
//             continued_fraction_real_gap_bound(coefficients, Nat.2 * k) <= Real.from_rat(iop(k))
//             continued_fraction_real_gap_bound_positive(coefficients, Nat.2 * k)
//             continued_fraction_real_gap_bound(coefficients, Nat.2 * k).is_positive
//             pos_gt_zero(continued_fraction_real_gap_bound(coefficients, Nat.2 * k))
//             Real.0 < continued_fraction_real_gap_bound(coefficients, Nat.2 * k)
//             lt_imp_lte(
//                 Real.0, continued_fraction_real_gap_bound(coefficients, Nat.2 * k))
//             Real.0 <= continued_fraction_real_gap_bound(coefficients, Nat.2 * k)
//             Real.0 <= continued_fraction_real_gap_bound(coefficients, Nat.2 * k) and
//                 continued_fraction_real_gap_bound(coefficients, Nat.2 * k) <= Real.from_rat(iop(k))
//         }
//         cf_converges_to_squeeze_nonneg(
//             function(k: Nat) {
//                 continued_fraction_real_gap_bound(coefficients, Nat.2 * k)
//             },
//             function(k: Nat) { Real.from_rat(iop(k)) })
//         continued_fraction_real_recip_suc_converges_to_zero
//         converges_to(
//             function(k: Nat) { Real.from_rat(iop(k)) },
//             Real.0)
//         converges_to(
//             function(k: Nat) {
//                 continued_fraction_real_gap_bound(coefficients, Nat.2 * k)
//             },
//             Real.0)
//     }
// }

/// Cancelling a common real term in a sum of differences.
theorem real_sub_add_cancel(a: Real, b: Real, c: Real) {
    (a - b) + (b - c) = a - c
} by {
    a - b = a + -b
    b - c = b + -c
    (a - b) + (b - c) = (a + -b) + (b + -c)
    (a + -b) + (b + -c) = a + -b + b + -c
    -b + b = Real.0
    a + -b + b + -c = a + (-b + b) + -c
    a + (-b + b) + -c = a + Real.0 + -c
    a + Real.0 + -c = a + -c
    a + -c = a - c
    (a - b) + (b - c) = a - c
}

/// For every positive tolerance, the convergent values are eventually within
/// it of the real limit.
theorem continued_fraction_convergents_tail_bound_for_eps(
    coefficients: Nat -> Nat, eps: Real
) {
    positive_continued_fraction_sequence_tail(coefficients) and eps.is_positive implies exists(n: Nat) {
            forall(m: Nat) {
                n <= m implies (continued_fraction_real_convergent_value(coefficients, m) - continued_fraction_real_limit(coefficients)).abs < eps
            }
        }
} by {
    if positive_continued_fraction_sequence_tail(coefficients) and eps.is_positive {
        eps_lt_half(eps)
        exists(eps2: Real) { eps2.is_positive and eps2 + eps2 < eps }
        let eps2: Real satisfy {
            eps2.is_positive and eps2 + eps2 < eps
        }
        continued_fraction_real_even_converges(coefficients)
        converges(continued_fraction_real_even_convergent(coefficients))
        convergent_converges_to_limit(continued_fraction_real_even_convergent(coefficients))
        converges_to(continued_fraction_real_even_convergent(coefficients),
            limit(continued_fraction_real_even_convergent(coefficients)))
        exists(n1: Nat) {
            tail_bound(continued_fraction_real_even_convergent(coefficients),
                limit(continued_fraction_real_even_convergent(coefficients)), n1, eps2)
        }
        let n1: Nat satisfy {
            tail_bound(continued_fraction_real_even_convergent(coefficients),
                limit(continued_fraction_real_even_convergent(coefficients)), n1, eps2)
        }
        pos_gt_zero(eps2)
        Real.0 < eps2
        rat_between_reals(Real.0, eps2)
        let r: Rat satisfy {
            Real.0 < Real.from_rat(r) and Real.from_rat(r) < eps2
        }
        rat_positive_of_real_positive(r)
        r.is_positive
        iop_gets_lt(r)
        let n2: Nat satisfy {
            forall(k: Nat) {
                n2 <= k implies iop(k) < r
            }
        }
        forall(m: Nat) {
            if Nat.2 * (n1 + n2) + Nat.1 <= m {
                nat_even_or_odd(m)
                if exists(k: Nat) { m = Nat.2 * k } {
                    let k: Nat satisfy {
                        m = Nat.2 * k
                    }
                    m = Nat.2 * k
                    Nat.2 * (n1 + n2) + Nat.1 <= Nat.2 * k
                    lt_suc(Nat.2 * (n1 + n2))
                    Nat.2 * (n1 + n2) < (Nat.2 * (n1 + n2)).suc
                    add_one_right(Nat.2 * (n1 + n2))
                    Nat.2 * (n1 + n2) + Nat.1 = (Nat.2 * (n1 + n2)).suc
                    Nat.2 * (n1 + n2) < Nat.2 * (n1 + n2) + Nat.1
                    lt_imp_lte(Nat.2 * (n1 + n2), Nat.2 * (n1 + n2) + Nat.1)
                    Nat.2 * (n1 + n2) <= Nat.2 * (n1 + n2) + Nat.1
                    lte_trans(Nat.2 * (n1 + n2), Nat.2 * (n1 + n2) + Nat.1, Nat.2 * k)
                    Nat.2 * (n1 + n2) <= Nat.2 * k
                    lte_double_imp_lte(k, n1 + n2)
                    n1 + n2 <= k
                    lte_add_right(n1, n1, n2)
                    n1 <= n1 + n2
                    lte_trans(n1, n1 + n2, k)
                    n1 <= k
                    tail_bound_implies_is_close(
                        continued_fraction_real_even_convergent(coefficients),
                        limit(continued_fraction_real_even_convergent(coefficients)),
                        n1, eps2, k)
                    continued_fraction_real_even_convergent(coefficients, k) .is_close(limit(continued_fraction_real_even_convergent(coefficients)), eps2)
                    (continued_fraction_real_even_convergent(coefficients, k) -
                        limit(continued_fraction_real_even_convergent(coefficients))).abs < eps2
                    continued_fraction_real_even_convergent(coefficients, k) =
                        continued_fraction_real_convergent_value(coefficients, m)
                    (continued_fraction_real_convergent_value(coefficients, m) -
                        continued_fraction_real_limit(coefficients)).abs < eps2
                    lt_add_right(Real.0, eps2, eps2)
                    Real.0 + eps2 < eps2 + eps2
                    Real.0 + eps2 = eps2
                    eps2 < eps2 + eps2
                    eps2 + eps2 < eps
                    lt_trans(eps2, eps2 + eps2, eps)
                    eps2 < eps
                    lt_trans(
                        (continued_fraction_real_convergent_value(coefficients, m) -
                            continued_fraction_real_limit(coefficients)).abs,
                        eps2, eps)
                    (continued_fraction_real_convergent_value(coefficients, m) -
                        continued_fraction_real_limit(coefficients)).abs < eps
                }
                if exists(k: Nat) { m = (Nat.2 * k).suc } {
                    let k: Nat satisfy {
                        m = (Nat.2 * k).suc
                    }
                    m = (Nat.2 * k).suc
                    Nat.2 * (n1 + n2) + Nat.1 <= (Nat.2 * k).suc
                    add_one_right(Nat.2 * k)
                    Nat.2 * k + Nat.1 = (Nat.2 * k).suc
                    Nat.2 * (n1 + n2) + Nat.1 <= Nat.2 * k + Nat.1
                    lte_cancel_suc(Nat.2 * (n1 + n2), Nat.2 * k)
                    Nat.2 * (n1 + n2) <= Nat.2 * k
                    lte_double_imp_lte(k, n1 + n2)
                    n1 + n2 <= k
                    lte_add_right(n1, n1, n2)
                    n1 <= n1 + n2
                    lte_trans(n1, n1 + n2, k)
                    n1 <= k
                    lte_add_left(n2, n2, n1)
                    n2 <= n1 + n2
                    lte_trans(n2, n1 + n2, k)
                    n2 <= k
                    tail_bound_implies_is_close(
                        continued_fraction_real_even_convergent(coefficients),
                        limit(continued_fraction_real_even_convergent(coefficients)),
                        n1, eps2, k)
                    continued_fraction_real_even_convergent(coefficients, k) .is_close(limit(continued_fraction_real_even_convergent(coefficients)), eps2)
                    (continued_fraction_real_even_convergent(coefficients, k) -
                        limit(continued_fraction_real_even_convergent(coefficients))).abs < eps2
                    iop(k) < r
                    from_rat_maintains_lt(iop(k), r)
                    Real.from_rat(iop(k)) < Real.from_rat(r)
                    Real.from_rat(r) < eps2
                    lt_trans(Real.from_rat(iop(k)), Real.from_rat(r), eps2)
                    Real.from_rat(iop(k)) < eps2
                    continued_fraction_real_gap_even_le_recip_suc(coefficients, k)
                    continued_fraction_real_gap_bound(coefficients, Nat.2 * k) <= Real.from_rat(iop(k))
                    real_lt_of_lte_of_lt(
                        continued_fraction_real_gap_bound(coefficients, Nat.2 * k),
                        Real.from_rat(iop(k)), eps2)
                    continued_fraction_real_gap_bound(coefficients, Nat.2 * k) < eps2
                    continued_fraction_real_gap_bound_positive(coefficients, Nat.2 * k)
                    continued_fraction_real_gap_bound(coefficients, Nat.2 * k).is_positive
                    pos_gt_zero(continued_fraction_real_gap_bound(coefficients, Nat.2 * k))
                    Real.0 < continued_fraction_real_gap_bound(coefficients, Nat.2 * k)
                    real_abs_of_nonneg(continued_fraction_real_gap_bound(coefficients, Nat.2 * k))
                    continued_fraction_real_gap_bound(coefficients, Nat.2 * k).abs =
                        continued_fraction_real_gap_bound(coefficients, Nat.2 * k)
                    continued_fraction_real_gap_bound(coefficients, Nat.2 * k).abs < eps2
                    continued_fraction_real_gap_eq_embedded_gap(coefficients, Nat.2 * k)
                    (continued_fraction_real_convergent_value(coefficients, (Nat.2 * k).suc) -
                        continued_fraction_real_convergent_value(coefficients, Nat.2 * k)).abs =
                        continued_fraction_real_gap_bound(coefficients, Nat.2 * k)
                    (continued_fraction_real_convergent_value(coefficients, (Nat.2 * k).suc) -
                        continued_fraction_real_convergent_value(coefficients, Nat.2 * k)).abs < eps2
                    continued_fraction_real_convergent_value(coefficients, m) =
                        continued_fraction_real_convergent_value(coefficients, (Nat.2 * k).suc)
                    continued_fraction_real_convergent_value(coefficients, Nat.2 * k) =
                        continued_fraction_real_even_convergent(coefficients, k)
                    real_sub_add_cancel(
                        continued_fraction_real_convergent_value(coefficients, m),
                        continued_fraction_real_even_convergent(coefficients, k),
                        continued_fraction_real_limit(coefficients))
                    (continued_fraction_real_convergent_value(coefficients, m) -
                            continued_fraction_real_even_convergent(coefficients, k)) +
                        (continued_fraction_real_even_convergent(coefficients, k) -
                            continued_fraction_real_limit(coefficients)) =
                        continued_fraction_real_convergent_value(coefficients, m) -
                        continued_fraction_real_limit(coefficients)
                    triangle_ineq(
                        continued_fraction_real_convergent_value(coefficients, m) -
                            continued_fraction_real_even_convergent(coefficients, k),
                        continued_fraction_real_even_convergent(coefficients, k) -
                            continued_fraction_real_limit(coefficients))
                    ((continued_fraction_real_convergent_value(coefficients, m) -
                            continued_fraction_real_even_convergent(coefficients, k)) +
                        (continued_fraction_real_even_convergent(coefficients, k) -
                            continued_fraction_real_limit(coefficients))).abs <= (continued_fraction_real_convergent_value(coefficients, m) - continued_fraction_real_even_convergent(coefficients, k)).abs + (continued_fraction_real_even_convergent(coefficients, k) - continued_fraction_real_limit(coefficients)).abs
                    (continued_fraction_real_convergent_value(coefficients, m) -
                        continued_fraction_real_limit(coefficients)).abs <= (continued_fraction_real_convergent_value(coefficients, m) - continued_fraction_real_even_convergent(coefficients, k)).abs + (continued_fraction_real_even_convergent(coefficients, k) - continued_fraction_real_limit(coefficients)).abs
                    (continued_fraction_real_convergent_value(coefficients, m) -
                        continued_fraction_real_even_convergent(coefficients, k)).abs < eps2
                    lt_add_right(
                        (continued_fraction_real_convergent_value(coefficients, m) -
                            continued_fraction_real_even_convergent(coefficients, k)).abs,
                        eps2,
                        (continued_fraction_real_even_convergent(coefficients, k) -
                            continued_fraction_real_limit(coefficients)).abs)
                    (continued_fraction_real_convergent_value(coefficients, m) -
                            continued_fraction_real_even_convergent(coefficients, k)).abs +
                        (continued_fraction_real_even_convergent(coefficients, k) -
                            continued_fraction_real_limit(coefficients)).abs < eps2 + (continued_fraction_real_even_convergent(coefficients, k) - continued_fraction_real_limit(coefficients)).abs
                    lt_add_left(
                        (continued_fraction_real_even_convergent(coefficients, k) -
                            continued_fraction_real_limit(coefficients)).abs,
                        eps2, eps2)
                    (continued_fraction_real_even_convergent(coefficients, k) -
                            continued_fraction_real_limit(coefficients)).abs + eps2 < eps2 + eps2
                    (continued_fraction_real_even_convergent(coefficients, k) -
                            continued_fraction_real_limit(coefficients)).abs < eps2
                    eps2 + (continued_fraction_real_even_convergent(coefficients, k) -
                            continued_fraction_real_limit(coefficients)).abs < eps2 + eps2
                    lt_trans(
                        (continued_fraction_real_convergent_value(coefficients, m) -
                            continued_fraction_real_even_convergent(coefficients, k)).abs +
                        (continued_fraction_real_even_convergent(coefficients, k) -
                            continued_fraction_real_limit(coefficients)).abs,
                        eps2 + (continued_fraction_real_even_convergent(coefficients, k) -
                            continued_fraction_real_limit(coefficients)).abs,
                        eps2 + eps2)
                    (continued_fraction_real_convergent_value(coefficients, m) -
                            continued_fraction_real_even_convergent(coefficients, k)).abs +
                        (continued_fraction_real_even_convergent(coefficients, k) -
                            continued_fraction_real_limit(coefficients)).abs < eps2 + eps2
                    real_lt_of_lte_of_lt(
                        (continued_fraction_real_convergent_value(coefficients, m) -
                            continued_fraction_real_limit(coefficients)).abs,
                        (continued_fraction_real_convergent_value(coefficients, m) -
                            continued_fraction_real_even_convergent(coefficients, k)).abs +
                        (continued_fraction_real_even_convergent(coefficients, k) -
                            continued_fraction_real_limit(coefficients)).abs,
                        eps2 + eps2)
                    (continued_fraction_real_convergent_value(coefficients, m) -
                        continued_fraction_real_limit(coefficients)).abs < eps2 + eps2
                    eps2 + eps2 < eps
                    lt_trans(
                        (continued_fraction_real_convergent_value(coefficients, m) -
                            continued_fraction_real_limit(coefficients)).abs,
                        eps2 + eps2, eps)
                    (continued_fraction_real_convergent_value(coefficients, m) -
                        continued_fraction_real_limit(coefficients)).abs < eps
                }
                (continued_fraction_real_convergent_value(coefficients, m) -
                    continued_fraction_real_limit(coefficients)).abs < eps
            }
        }
        exists(n: Nat) {
            forall(m: Nat) {
                n <= m implies (continued_fraction_real_convergent_value(coefficients, m) - continued_fraction_real_limit(coefficients)).abs < eps
            }
        }
    }
}

// The convergents converge to the real limit in the library's `converges_to`
// formulation.  The epsilon-N content of this statement is proved above as
// `continued_fraction_convergents_tail_bound_for_eps`; wrapping it in the
// library's `converges_to` predicate is left to a file inside the real
// package, because the final forall-eps closure is not reachable from outside
// the package within the search limits.
//
// theorem continued_fraction_convergents_converge_to_limit(
//     coefficients: Nat -> Nat
// ) {
//     positive_continued_fraction_sequence_tail(coefficients) implies
//         converges_to(continued_fraction_real_convergent_value(coefficients),
//             continued_fraction_real_limit(coefficients))
// } by {
//     if positive_continued_fraction_sequence_tail(coefficients) {
//         continued_fraction_convergents_tail_bound_for_eps(coefficients)
//     }
// }
//
// theorem continued_fraction_convergents_converge(coefficients: Nat -> Nat) {
//     positive_continued_fraction_sequence_tail(coefficients) implies
//         converges(continued_fraction_real_convergent_value(coefficients))
// } by {
//     if positive_continued_fraction_sequence_tail(coefficients) {
//         continued_fraction_convergents_converge_to_limit(coefficients)
//         converges_to_imp_converges(
//             continued_fraction_real_convergent_value(coefficients),
//             continued_fraction_real_limit(coefficients))
//     }
// }
