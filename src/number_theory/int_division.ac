from nat import Nat, div_mod_decomp, mod_lt, div_of_decomp, mod_of_decomp,
    div_imp_mod, divides_trans, mul_comm, add_zero_right, add_suc_right,
    lte_ref, lte_trans, lte_add_right, succ_div_yes, succ_div_no
from number_theory.congruence import mod_mul_eq
numerals Nat

/// The division algorithm: if `a = q * m + r` with a remainder strictly below
/// the modulus, then `q` is the quotient and `r` is the remainder of `a` by `m`.
/// This restates the quotient-remainder uniqueness theorems of
/// `nat/division.ac` in terms of an arbitrary decomposition of `a`.
theorem division_algorithm(a: Nat, q: Nat, r: Nat, m: Nat) {
    a = q * m + r and r < m implies a.div(m) = q and a.mod(m) = r
} by {
    if a = q * m + r and r < m {
        a = q * m + r
        r < m
        div_of_decomp(q, r, m)
        (q * m + r).div(m) = q
        a.div(m) = q
        mod_of_decomp(q, r, m)
        (q * m + r).mod(m) = r
        a.mod(m) = r
        a.div(m) = q and a.mod(m) = r
    }
}

/// Divisibility via the remainder: `m` divides `a` exactly when the remainder
/// of `a` modulo `m` is zero. The forward direction is `div_imp_mod`; the
/// converse follows from the division decomposition.
theorem divides_iff_mod_zero(a: Nat, m: Nat) {
    m.divides(a) = (a.mod(m) = Nat.0)
} by {
    if m.divides(a) {
        div_imp_mod(a, m)
        a.mod(m) = Nat.0
    }
    if a.mod(m) = Nat.0 {
        div_mod_decomp(a, m)
        a.div(m) * m + a.mod(m) = a
        a.div(m) * m + Nat.0 = a
        add_zero_right(a.div(m) * m)
        a.div(m) * m + Nat.0 = a.div(m) * m
        a.div(m) * m = a
        mul_comm(a.div(m), m)
        m * a.div(m) = a
        m.divides(a) = exists(c: Nat) { m * c = a }
        exists(c: Nat) { m * c = a }
        m.divides(a)
    }
}

/// Transitivity of divisibility: if `a` divides `b` and `b` divides `c`, then
/// `a` divides `c`. This restates `divides_trans` from `nat`.
theorem divisibility_trans(a: Nat, b: Nat, c: Nat) {
    a.divides(b) and b.divides(c) implies a.divides(c)
} by {
    if a.divides(b) and b.divides(c) {
        divides_trans(a, b, c)
        a.divides(c)
    }
}

/// Modular multiplication: the remainder of a product modulo `m` depends only
/// on the remainders of the factors. This restates `mod_mul_eq` from
/// `number_theory/congruence.ac`.
theorem mod_mul_components(a: Nat, b: Nat, m: Nat) {
    (a * b).mod(m) = (a.mod(m) * b.mod(m)).mod(m)
} by {
    mod_mul_eq(a, b, m)
    (a * b).mod(m) = (a.mod(m) * b.mod(m)).mod(m)
}

/// Monotonicity of division: adding a natural number to the dividend never
/// decreases the quotient. Proved by induction on the added number, using the
/// successor-quotient recurrences `succ_div_yes` and `succ_div_no`.
theorem div_add_ge_left(a: Nat, b: Nat, m: Nat) {
    m != Nat.0 implies a.div(m) <= (a + b).div(m)
} by {
    if m != Nat.0 {
        let f: Nat -> Bool = function(x: Nat) {
            a.div(m) <= (a + x).div(m)
        }
        add_zero_right(a)
        a + Nat.0 = a
        lte_ref(a.div(m))
        a.div(m) <= a.div(m)
        a.div(m) <= (a + Nat.0).div(m)
        f(Nat.0)
        forall(x: Nat) {
            if f(x) {
                a.div(m) <= (a + x).div(m)
                add_suc_right(a, x)
                a + x.suc = (a + x).suc
                if m.divides((a + x).suc) {
                    succ_div_yes(a + x, m)
                    (a + x).suc.div(m) = (a + x).div(m) + Nat.1
                    (a + x).div(m) <= (a + x).div(m) + Nat.1
                    (a + x).div(m) <= (a + x).suc.div(m)
                    lte_trans(a.div(m), (a + x).div(m), (a + x).suc.div(m))
                    a.div(m) <= (a + x).suc.div(m)
                    a.div(m) <= (a + x.suc).div(m)
                    f(x.suc)
                }
                if not m.divides((a + x).suc) {
                    succ_div_no(a + x, m)
                    (a + x).suc.div(m) = (a + x).div(m)
                    lte_trans(a.div(m), (a + x).div(m), (a + x).suc.div(m))
                    a.div(m) <= (a + x).suc.div(m)
                    a.div(m) <= (a + x.suc).div(m)
                    f(x.suc)
                }
                f(x.suc)
            }
        }
        f(b)
        a.div(m) <= (a + b).div(m)
    }
}
