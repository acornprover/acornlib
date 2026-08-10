// Graham's number and Knuth's up-arrow notation.
//
// Knuth's up-arrow notation is the hierarchy of increasingly fast-growing
// operations on the natural numbers:
//
//   a ↑^0 b = a^b                          (exponentiation)
//   a ↑^(n+1) b = a ↑^n (a ↑^(n+1) (b-1))  (iterated application, b times)
//
// so that a ↑↑ b (tetration) is an exponential tower of height b, a ↑↑↑ b is
// an iterated tower, and so on.  Graham's number is the famous upper bound
// from Ramsey theory (Graham & Rothschild 1971, popularized by Martin
// Gardner): the smallest dimension n such that every 2-colouring of the
// 2-dimensional hyperedges of an n-dimensional hypercube contains a
// monochromatic planar K_4.  It is defined by the recursion
//
//   g_1 = 3 ↑↑↑↑ 3,     g_{k+1} = 3 ↑^{g_k} 3,     G = g_64.
//
// This file develops the foundations:
//
//   (a) the general up-arrow `up_arrow(a, n, b)` as iterated function
//       application, with its defining recursion law `up_arrow_suc`;
//   (b) the materialized low levels: `tetration` (a ↑↑ b) and `up_arrow3`
//       (a ↑↑↑ b), with the bridge theorems connecting them to the general
//       arrow;
//   (c) the small tower values: 3 ↑↑ 2 = 27, 3 ↑↑ 3 = 3^27, 2 ↑↑ 3 = 16,
//       2 ↑↑ 4 = 65536, and 3 ↑↑↑ 2 = 3 ↑↑ 3 = 3^27 (targets a, b, c);
//   (d) the growth statement 2 ↑↑ 5 = 2^65536 (target d);
//   (e) Graham's number G = g_64 as a defined-but-not-computable value
//       (target e; the recursion and the base value are stated, the tower
//       itself is beyond computation).

from nat import Nat, alt_induction
from nat import exp_add, exp_mul, exp_one, exp_zero, sq_eq_mul,
    mul_comm, mul_assoc, mul_one_left, mul_two_left,
    read_add_single, read_add_read, read_mul_single, read_read_carry,
    nat_mul_2_5, nat_mul_5_5, nat_mul_6_5, nat_mul_6_6, nat_mul_25_6,
    nat_mul_3_3, nat_mul_9_3

numerals Nat

// ---------------------------------------------------------------------------
// Function iteration: the engine behind every arrow level.
// ---------------------------------------------------------------------------

/// Applies the function f exactly b times, starting from start.
define iter_nat(f: Nat -> Nat, b: Nat, start: Nat) -> Nat {
    match b {
        Nat.zero {
            start
        }
        Nat.suc(m) {
            iter_nat(f, m, f(start))
        }
    }
}

/// Zero iterations leave the start value unchanged.
theorem iter_nat_zero(f: Nat -> Nat, s: Nat) {
    iter_nat(f, Nat.0, s) = s
}

/// A successor iteration applies f once and iterates the result b more times.
theorem iter_nat_suc_step(f: Nat -> Nat, b: Nat, s: Nat) {
    iter_nat(f, b.suc, s) = iter_nat(f, b, f(s))
}

/// Iteration commutes with applying the iterated function: b iterations of f
/// after one application of f equal one application of f after b iterations.
theorem iter_nat_commute(f: Nat -> Nat, b: Nat, x: Nat) {
    iter_nat(f, b, f(x)) = f(iter_nat(f, b, x))
} by {
    define g(m: Nat) -> Bool {
        forall(z: Nat) { iter_nat(f, m, f(z)) = f(iter_nat(f, m, z)) }
    }
    forall(z: Nat) {
        iter_nat(f, Nat.0, f(z)) = f(z)
        f(iter_nat(f, Nat.0, z)) = f(z)
        iter_nat(f, Nat.0, f(z)) = f(iter_nat(f, Nat.0, z))
    }
    g(Nat.0)
    forall(m: Nat) {
        if g(m) {
            g(m) = forall(z: Nat) { iter_nat(f, m, f(z)) = f(iter_nat(f, m, z)) }
            forall(z: Nat) {
                iter_nat(f, m, f(z)) = f(iter_nat(f, m, z))
                iter_nat(f, m.suc, f(z)) = iter_nat(f, m, f(f(z)))
                iter_nat(f, m, f(f(z))) = f(iter_nat(f, m, f(z)))
                f(iter_nat(f, m, f(z))) = f(f(iter_nat(f, m, z)))
                f(f(iter_nat(f, m, z))) = f(iter_nat(f, m.suc, z))
                iter_nat(f, m.suc, f(z)) = f(iter_nat(f, m.suc, z))
            }
            g(m.suc)
        }
    }
    alt_induction(g)
    g(b)
    g(b) = forall(z: Nat) { iter_nat(f, b, f(z)) = f(iter_nat(f, b, z)) }
    iter_nat(f, b, f(x)) = f(iter_nat(f, b, x))
}

// ---------------------------------------------------------------------------
// Knuth's up-arrow, general level.
// ---------------------------------------------------------------------------

/// Knuth's up-arrow: `up_arrow(a, n, b)` is `a ↑^n b`.  Level zero is
/// exponentiation; level `n + 1` iterates the level-n arrow `b` times from 1,
/// which is the recursion `a ↑^(n+1) b = a ↑^n (a ↑^(n+1) (b-1))`.
define up_arrow(a: Nat, n: Nat, b: Nat) -> Nat {
    match n {
        Nat.zero {
            a.pow(b)
        }
        Nat.suc(k) {
            iter_nat(function(x: Nat) { up_arrow(a, k, x) }, b, Nat.1)
        }
    }
}

/// Applying a level-k arrow inside a function literal is the arrow itself.
theorem up_arrow_beta(a: Nat, k: Nat, x: Nat) {
    (function(y: Nat) { up_arrow(a, k, y) })(x) = up_arrow(a, k, x)
}

/// Level zero of the up-arrow is exponentiation: `a ↑^0 b = a^b`.
theorem up_arrow_exp(a: Nat, b: Nat) {
    up_arrow(a, Nat.0, b) = a.pow(b)
}

/// Every level above zero applied to zero arguments is one.
theorem up_arrow_zero(a: Nat, n: Nat) {
    up_arrow(a, n.suc, Nat.0) = Nat.1
}

/// The defining recursion of the up-arrow:
/// `a ↑^(n+1) (b+1) = a ↑^n (a ↑^(n+1) b)`.
theorem up_arrow_suc(a: Nat, k: Nat, m: Nat) {
    up_arrow(a, k.suc, m.suc) = up_arrow(a, k, up_arrow(a, k.suc, m))
} by {
    up_arrow(a, k.suc, m.suc) = iter_nat(function(y: Nat) { up_arrow(a, k, y) }, m.suc, Nat.1)
    iter_nat(function(y: Nat) { up_arrow(a, k, y) }, m.suc, Nat.1) = iter_nat(function(y: Nat) { up_arrow(a, k, y) }, m, up_arrow(a, k, Nat.1))
    up_arrow_beta(a, k, Nat.1)
    iter_nat(function(y: Nat) { up_arrow(a, k, y) }, m, up_arrow(a, k, Nat.1)) = iter_nat(function(y: Nat) { up_arrow(a, k, y) }, m, (function(y: Nat) { up_arrow(a, k, y) })(Nat.1))
    iter_nat_commute(function(y: Nat) { up_arrow(a, k, y) }, m, Nat.1)
    iter_nat(function(y: Nat) { up_arrow(a, k, y) }, m, (function(y: Nat) { up_arrow(a, k, y) })(Nat.1)) = (function(y: Nat) { up_arrow(a, k, y) })(iter_nat(function(y: Nat) { up_arrow(a, k, y) }, m, Nat.1))
    up_arrow_beta(a, k, iter_nat(function(y: Nat) { up_arrow(a, k, y) }, m, Nat.1))
    (function(y: Nat) { up_arrow(a, k, y) })(iter_nat(function(y: Nat) { up_arrow(a, k, y) }, m, Nat.1)) = up_arrow(a, k, iter_nat(function(y: Nat) { up_arrow(a, k, y) }, m, Nat.1))
    up_arrow(a, k.suc, m) = iter_nat(function(y: Nat) { up_arrow(a, k, y) }, m, Nat.1)
    up_arrow(a, k, iter_nat(function(y: Nat) { up_arrow(a, k, y) }, m, Nat.1)) = up_arrow(a, k, up_arrow(a, k.suc, m))
    up_arrow(a, k.suc, m.suc) = up_arrow(a, k, up_arrow(a, k.suc, m))
}

// ---------------------------------------------------------------------------
// The low levels, materialized: tetration and the third arrow.
// ---------------------------------------------------------------------------

/// Tetration: `tetration(a, b)` is `a ↑↑ b`, the exponential tower with b
/// copies of a: `a^(a^(...^a))`.
define tetration(a: Nat, b: Nat) -> Nat {
    match b {
        Nat.zero {
            Nat.1
        }
        Nat.suc(m) {
            a.pow(tetration(a, m))
        }
    }
}

/// The empty tower is one.
theorem tetration_zero(a: Nat) {
    tetration(a, Nat.0) = Nat.1
}

/// A tower of height b + 1 is a raised to the tower of height b.
theorem tetration_suc(a: Nat, m: Nat) {
    tetration(a, m.suc) = a.pow(tetration(a, m))
}

/// The single-arrow level of the general up-arrow is exactly tetration:
/// `a ↑^1 b = a ↑↑ b`.
theorem up_arrow_bridge_one(a: Nat, b: Nat) {
    up_arrow(a, Nat.1, b) = tetration(a, b)
} by {
    define h(n: Nat) -> Bool { up_arrow(a, Nat.1, n) = tetration(a, n) }
    up_arrow(a, Nat.1, Nat.0) = iter_nat(function(y: Nat) { up_arrow(a, Nat.0, y) }, Nat.0, Nat.1)
    iter_nat(function(y: Nat) { up_arrow(a, Nat.0, y) }, Nat.0, Nat.1) = Nat.1
    tetration(a, Nat.0) = Nat.1
    up_arrow(a, Nat.1, Nat.0) = tetration(a, Nat.0)
    h(Nat.0)
    forall(n: Nat) {
        if h(n) {
            up_arrow(a, Nat.1, n) = tetration(a, n)
            up_arrow(a, Nat.1, n.suc) = up_arrow(a, Nat.0, up_arrow(a, Nat.1, n))
            up_arrow(a, Nat.0, up_arrow(a, Nat.1, n)) = a.pow(up_arrow(a, Nat.1, n))
            a.pow(up_arrow(a, Nat.1, n)) = a.pow(tetration(a, n))
            a.pow(tetration(a, n)) = tetration(a, n.suc)
            up_arrow(a, Nat.1, n.suc) = tetration(a, n.suc)
            h(n.suc)
        }
    }
    alt_induction(h)
    h(b)
    up_arrow(a, Nat.1, b) = tetration(a, b)
}

/// The third arrow: `up_arrow3(a, b)` is `a ↑↑↑ b`, the function that
/// iterates tetration b times, starting from one.
define up_arrow3(a: Nat, b: Nat) -> Nat {
    match b {
        Nat.zero {
            Nat.1
        }
        Nat.suc(m) {
            tetration(a, up_arrow3(a, m))
        }
    }
}

/// Zero iterations of tetration give one.
theorem up_arrow3_zero(a: Nat) {
    up_arrow3(a, Nat.0) = Nat.1
}

/// A successor iteration of the third arrow applies tetration once more.
theorem up_arrow3_suc(a: Nat, m: Nat) {
    up_arrow3(a, m.suc) = tetration(a, up_arrow3(a, m))
}

/// The double-arrow level of the general up-arrow is the third arrow:
/// `a ↑^2 b = a ↑↑↑ b`.
theorem up_arrow_bridge_two(a: Nat, b: Nat) {
    up_arrow(a, Nat.2, b) = up_arrow3(a, b)
} by {
    define h(n: Nat) -> Bool { up_arrow(a, Nat.2, n) = up_arrow3(a, n) }
    up_arrow(a, Nat.2, Nat.0) = Nat.1
    up_arrow3(a, Nat.0) = Nat.1
    up_arrow(a, Nat.2, Nat.0) = up_arrow3(a, Nat.0)
    h(Nat.0)
    forall(n: Nat) {
        if h(n) {
            up_arrow(a, Nat.2, n) = up_arrow3(a, n)
            up_arrow(a, Nat.2, n.suc) = up_arrow(a, Nat.1, up_arrow(a, Nat.2, n))
            up_arrow(a, Nat.1, up_arrow(a, Nat.2, n)) = tetration(a, up_arrow(a, Nat.2, n))
            tetration(a, up_arrow(a, Nat.2, n)) = tetration(a, up_arrow3(a, n))
            tetration(a, up_arrow3(a, n)) = up_arrow3(a, n.suc)
            up_arrow(a, Nat.2, n.suc) = up_arrow3(a, n.suc)
            h(n.suc)
        }
    }
    alt_induction(h)
    h(b)
    up_arrow(a, Nat.2, b) = up_arrow3(a, b)
}

// ---------------------------------------------------------------------------
// Small powers and the two-digit products behind 2^16 = 65536.
// ---------------------------------------------------------------------------

/// `3^3 = 27`.
theorem pow_3_3_value {
    Nat.3.pow(Nat.3) = Nat.27
} by {
    Nat.3.pow(Nat.3) = Nat.3.pow(Nat.2 + Nat.1)
    exp_add(Nat.3, Nat.2, Nat.1)
    Nat.3.pow(Nat.2 + Nat.1) = Nat.3.pow(Nat.2) * Nat.3.pow(Nat.1)
    sq_eq_mul(Nat.3)
    exp_one(Nat.3)
    Nat.3.pow(Nat.2) * Nat.3.pow(Nat.1) = (Nat.3 * Nat.3) * Nat.3
    Nat.3 * Nat.3 = Nat.9
    (Nat.3 * Nat.3) * Nat.3 = Nat.9 * Nat.3
    Nat.9 * Nat.3 = Nat.27
}

/// `2^2 = 4`.
theorem pow_2_2_value {
    Nat.2.pow(Nat.2) = Nat.4
} by {
    sq_eq_mul(Nat.2)
    Nat.2.pow(Nat.2) = Nat.2 * Nat.2
    Nat.2 * Nat.2 = Nat.4
}

/// `2^4 = 16`.
theorem pow_2_4_value {
    Nat.2.pow(Nat.4) = Nat.16
} by {
    Nat.2.pow(Nat.4) = Nat.2.pow(Nat.2 + Nat.2)
    exp_add(Nat.2, Nat.2, Nat.2)
    Nat.2.pow(Nat.2 + Nat.2) = Nat.2.pow(Nat.2) * Nat.2.pow(Nat.2)
    pow_2_2_value
    Nat.2.pow(Nat.2) = Nat.4
    Nat.2.pow(Nat.2) * Nat.2.pow(Nat.2) = Nat.4 * Nat.4
    Nat.4 * Nat.4 = Nat.16
}

/// `2^5 = 32`, by doubling `2^4`.
theorem pow_2_5_value {
    Nat.2.pow(Nat.5) = Nat.32
} by {
    Nat.2.pow(Nat.5) = Nat.2.pow(Nat.4 + Nat.1)
    exp_add(Nat.2, Nat.4, Nat.1)
    Nat.2.pow(Nat.4 + Nat.1) = Nat.2.pow(Nat.4) * Nat.2.pow(Nat.1)
    exp_one(Nat.2)
    Nat.2.pow(Nat.1) = Nat.2
    Nat.4 + Nat.1 = Nat.5
    Nat.2.pow(Nat.5) = Nat.2.pow(Nat.4) * Nat.2
    pow_2_4_value
    Nat.2.pow(Nat.4) = Nat.16
    Nat.2.pow(Nat.4) * Nat.2 = Nat.16 * Nat.2
    mul_comm(Nat.16, Nat.2)
    Nat.16 * Nat.2 = Nat.2 * Nat.16
    mul_two_left(Nat.16)
    Nat.2 * Nat.16 = Nat.16 + Nat.16
    Nat.16 = Nat.1.read(Nat.6)
    read_add_read(Nat.1, Nat.6, Nat.1, Nat.6)
    Nat.1.read(Nat.6) + Nat.1.read(Nat.6) = (Nat.1 + Nat.1).read(Nat.6 + Nat.6)
    Nat.1 + Nat.1 = Nat.2
    Nat.6 + Nat.6 = Nat.12
    (Nat.1 + Nat.1).read(Nat.6 + Nat.6) = Nat.2.read(Nat.12)
    Nat.2.read(Nat.12) = Nat.32
    Nat.16 + Nat.16 = Nat.32
    Nat.2.pow(Nat.5) = Nat.32
}

/// Doubling `32` gives `64`.
theorem nat_mul_2_32 {
    Nat.2 * Nat.32 = Nat.64
} by {
    mul_two_left(Nat.32)
    Nat.2 * Nat.32 = Nat.32 + Nat.32
    Nat.32 = Nat.3.read(Nat.2)
    read_add_read(Nat.3, Nat.2, Nat.3, Nat.2)
    Nat.3.read(Nat.2) + Nat.3.read(Nat.2) = (Nat.3 + Nat.3).read(Nat.2 + Nat.2)
    Nat.3 + Nat.3 = Nat.6
    Nat.2 + Nat.2 = Nat.4
    (Nat.3 + Nat.3).read(Nat.2 + Nat.2) = Nat.6.read(Nat.4)
    Nat.6.read(Nat.4) = Nat.64
    Nat.32 + Nat.32 = Nat.64
    Nat.2 * Nat.32 = Nat.64
}

/// Doubling `64` gives `128`.
theorem nat_mul_2_64 {
    Nat.2 * Nat.64 = Nat.128
} by {
    mul_two_left(Nat.64)
    Nat.2 * Nat.64 = Nat.64 + Nat.64
    Nat.64 = Nat.6.read(Nat.4)
    read_add_read(Nat.6, Nat.4, Nat.6, Nat.4)
    Nat.6.read(Nat.4) + Nat.6.read(Nat.4) = (Nat.6 + Nat.6).read(Nat.4 + Nat.4)
    Nat.6 + Nat.6 = Nat.12
    Nat.4 + Nat.4 = Nat.8
    (Nat.6 + Nat.6).read(Nat.4 + Nat.4) = Nat.12.read(Nat.8)
    Nat.12.read(Nat.8) = Nat.128
    Nat.64 + Nat.64 = Nat.128
    Nat.2 * Nat.64 = Nat.128
}

/// Doubling `128` gives `256`.
theorem nat_mul_2_128 {
    Nat.2 * Nat.128 = Nat.256
} by {
    mul_two_left(Nat.128)
    Nat.2 * Nat.128 = Nat.128 + Nat.128
    Nat.128 = Nat.12.read(Nat.8)
    read_add_read(Nat.12, Nat.8, Nat.12, Nat.8)
    Nat.12.read(Nat.8) + Nat.12.read(Nat.8) = (Nat.12 + Nat.12).read(Nat.8 + Nat.8)
    Nat.12 = Nat.1.read(Nat.2)
    read_add_read(Nat.1, Nat.2, Nat.1, Nat.2)
    Nat.1.read(Nat.2) + Nat.1.read(Nat.2) = (Nat.1 + Nat.1).read(Nat.2 + Nat.2)
    Nat.1 + Nat.1 = Nat.2
    Nat.2 + Nat.2 = Nat.4
    (Nat.1 + Nat.1).read(Nat.2 + Nat.2) = Nat.2.read(Nat.4)
    Nat.2.read(Nat.4) = Nat.24
    Nat.12 + Nat.12 = Nat.24
    Nat.8 + Nat.8 = Nat.16
    (Nat.12 + Nat.12).read(Nat.8 + Nat.8) = Nat.24.read(Nat.16)
    Nat.24.read(Nat.16) = Nat.24.read(Nat.10 * Nat.1 + Nat.6)
    read_read_carry(Nat.24, Nat.1, Nat.6)
    Nat.24.read(Nat.10 * Nat.1 + Nat.6) = (Nat.24 + Nat.1).read(Nat.6)
    Nat.24 + Nat.1 = Nat.25
    (Nat.24 + Nat.1).read(Nat.6) = Nat.25.read(Nat.6)
    Nat.25.read(Nat.6) = Nat.256
    Nat.128 + Nat.128 = Nat.256
    Nat.2 * Nat.128 = Nat.256
}

/// Doubling `256` gives `512`.
theorem nat_mul_2_256 {
    Nat.2 * Nat.256 = Nat.512
} by {
    mul_two_left(Nat.256)
    Nat.2 * Nat.256 = Nat.256 + Nat.256
    Nat.256 = Nat.25.read(Nat.6)
    read_add_read(Nat.25, Nat.6, Nat.25, Nat.6)
    Nat.25.read(Nat.6) + Nat.25.read(Nat.6) = (Nat.25 + Nat.25).read(Nat.6 + Nat.6)
    Nat.25 = Nat.2.read(Nat.5)
    read_add_read(Nat.2, Nat.5, Nat.2, Nat.5)
    Nat.2.read(Nat.5) + Nat.2.read(Nat.5) = (Nat.2 + Nat.2).read(Nat.5 + Nat.5)
    Nat.2 + Nat.2 = Nat.4
    Nat.5 + Nat.5 = Nat.10
    (Nat.2 + Nat.2).read(Nat.5 + Nat.5) = Nat.4.read(Nat.10)
    Nat.4.read(Nat.10) = Nat.4.read(Nat.10 * Nat.1 + Nat.0)
    read_read_carry(Nat.4, Nat.1, Nat.0)
    Nat.4.read(Nat.10 * Nat.1 + Nat.0) = (Nat.4 + Nat.1).read(Nat.0)
    Nat.4 + Nat.1 = Nat.5
    (Nat.4 + Nat.1).read(Nat.0) = Nat.5.read(Nat.0)
    Nat.5.read(Nat.0) = Nat.50
    Nat.25 + Nat.25 = Nat.50
    Nat.6 + Nat.6 = Nat.12
    (Nat.25 + Nat.25).read(Nat.6 + Nat.6) = Nat.50.read(Nat.12)
    Nat.50.read(Nat.12) = Nat.50.read(Nat.10 * Nat.1 + Nat.2)
    read_read_carry(Nat.50, Nat.1, Nat.2)
    Nat.50.read(Nat.10 * Nat.1 + Nat.2) = (Nat.50 + Nat.1).read(Nat.2)
    Nat.50 + Nat.1 = Nat.51
    (Nat.50 + Nat.1).read(Nat.2) = Nat.51.read(Nat.2)
    Nat.51.read(Nat.2) = Nat.512
    Nat.256 + Nat.256 = Nat.512
    Nat.2 * Nat.256 = Nat.512
}

/// `25 * 5 = 125`.
theorem nat_mul_25_5 {
    Nat.25 * Nat.5 = Nat.125
} by {
    Nat.25 = Nat.2.read(Nat.5)
    read_mul_single(Nat.2, Nat.5, Nat.5)
    Nat.2.read(Nat.5) * Nat.5 = (Nat.2 * Nat.5).read(Nat.5 * Nat.5)
    Nat.2 * Nat.5 = Nat.10
    Nat.5 * Nat.5 = Nat.25
    Nat.10.read(Nat.25) = Nat.10.read(Nat.10 * Nat.2 + Nat.5)
    read_read_carry(Nat.10, Nat.2, Nat.5)
    Nat.10.read(Nat.10 * Nat.2 + Nat.5) = (Nat.10 + Nat.2).read(Nat.5)
    Nat.10 + Nat.2 = Nat.12
    (Nat.10 + Nat.2).read(Nat.5) = Nat.12.read(Nat.5)
    Nat.12.read(Nat.5) = Nat.125
    Nat.25 * Nat.5 = Nat.125
}

/// `125 + 3 = 128`.
theorem nat_add_125_3 {
    Nat.125 + Nat.3 = Nat.128
} by {
    Nat.125 = Nat.12.read(Nat.5)
    read_add_single(Nat.12, Nat.5, Nat.3)
    Nat.12.read(Nat.5) + Nat.3 = Nat.12.read(Nat.5 + Nat.3)
    Nat.5 + Nat.3 = Nat.8
    Nat.12.read(Nat.5 + Nat.3) = Nat.12.read(Nat.8)
    Nat.12.read(Nat.8) = Nat.128
    Nat.125 + Nat.3 = Nat.128
}

/// `5 * 256 = 1280`.
theorem nat_mul_5_256 {
    Nat.5 * Nat.256 = Nat.1280
} by {
    mul_comm(Nat.5, Nat.256)
    Nat.5 * Nat.256 = Nat.256 * Nat.5
    Nat.256 = Nat.25.read(Nat.6)
    read_mul_single(Nat.25, Nat.6, Nat.5)
    Nat.25.read(Nat.6) * Nat.5 = (Nat.25 * Nat.5).read(Nat.6 * Nat.5)
    nat_mul_25_5
    Nat.25 * Nat.5 = Nat.125
    Nat.6 * Nat.5 = Nat.30
    Nat.125.read(Nat.30) = Nat.125.read(Nat.10 * Nat.3 + Nat.0)
    read_read_carry(Nat.125, Nat.3, Nat.0)
    Nat.125.read(Nat.10 * Nat.3 + Nat.0) = (Nat.125 + Nat.3).read(Nat.0)
    nat_add_125_3
    Nat.125 + Nat.3 = Nat.128
    (Nat.125 + Nat.3).read(Nat.0) = Nat.128.read(Nat.0)
    Nat.128.read(Nat.0) = Nat.1280
    Nat.5 * Nat.256 = Nat.1280
}

/// `512 + 128 = 640`.
theorem nat_add_512_128 {
    Nat.512 + Nat.128 = Nat.640
} by {
    Nat.512 = Nat.51.read(Nat.2)
    Nat.128 = Nat.12.read(Nat.8)
    read_add_read(Nat.51, Nat.2, Nat.12, Nat.8)
    Nat.51.read(Nat.2) + Nat.12.read(Nat.8) = (Nat.51 + Nat.12).read(Nat.2 + Nat.8)
    Nat.51 + Nat.12 = Nat.63
    Nat.2 + Nat.8 = Nat.10
    (Nat.51 + Nat.12).read(Nat.2 + Nat.8) = Nat.63.read(Nat.10)
    Nat.63.read(Nat.10) = Nat.63.read(Nat.10 * Nat.1 + Nat.0)
    read_read_carry(Nat.63, Nat.1, Nat.0)
    Nat.63.read(Nat.10 * Nat.1 + Nat.0) = (Nat.63 + Nat.1).read(Nat.0)
    Nat.63 + Nat.1 = Nat.64
    (Nat.63 + Nat.1).read(Nat.0) = Nat.64.read(Nat.0)
    Nat.64.read(Nat.0) = Nat.640
    Nat.512 + Nat.128 = Nat.640
}

/// `25 * 256 = 6400`.
theorem nat_mul_25_256 {
    Nat.25 * Nat.256 = Nat.6400
} by {
    Nat.25 = Nat.2.read(Nat.5)
    read_mul_single(Nat.2, Nat.5, Nat.256)
    Nat.2.read(Nat.5) * Nat.256 = (Nat.2 * Nat.256).read(Nat.5 * Nat.256)
    nat_mul_2_256
    Nat.2 * Nat.256 = Nat.512
    nat_mul_5_256
    Nat.5 * Nat.256 = Nat.1280
    Nat.512.read(Nat.1280) = Nat.512.read(Nat.10 * Nat.128 + Nat.0)
    read_read_carry(Nat.512, Nat.128, Nat.0)
    Nat.512.read(Nat.10 * Nat.128 + Nat.0) = (Nat.512 + Nat.128).read(Nat.0)
    nat_add_512_128
    Nat.512 + Nat.128 = Nat.640
    (Nat.512 + Nat.128).read(Nat.0) = Nat.640.read(Nat.0)
    Nat.640.read(Nat.0) = Nat.6400
    Nat.25 * Nat.256 = Nat.6400
}

/// `6 * 256 = 1536`.
theorem nat_mul_6_256 {
    Nat.6 * Nat.256 = Nat.1536
} by {
    mul_comm(Nat.6, Nat.256)
    Nat.6 * Nat.256 = Nat.256 * Nat.6
    Nat.256 = Nat.25.read(Nat.6)
    read_mul_single(Nat.25, Nat.6, Nat.6)
    Nat.25.read(Nat.6) * Nat.6 = (Nat.25 * Nat.6).read(Nat.6 * Nat.6)
    Nat.25 * Nat.6 = Nat.150
    Nat.6 * Nat.6 = Nat.36
    Nat.150.read(Nat.36) = Nat.150.read(Nat.10 * Nat.3 + Nat.6)
    read_read_carry(Nat.150, Nat.3, Nat.6)
    Nat.150.read(Nat.10 * Nat.3 + Nat.6) = (Nat.150 + Nat.3).read(Nat.6)
    Nat.150 + Nat.3 = Nat.153
    (Nat.150 + Nat.3).read(Nat.6) = Nat.153.read(Nat.6)
    Nat.153.read(Nat.6) = Nat.1536
    Nat.6 * Nat.256 = Nat.1536
}

/// `6400 + 153 = 6553`.
theorem nat_add_6400_153 {
    Nat.6400 + Nat.153 = Nat.6553
} by {
    Nat.6400 = Nat.640.read(Nat.0)
    Nat.153 = Nat.15.read(Nat.3)
    read_add_read(Nat.640, Nat.0, Nat.15, Nat.3)
    Nat.640.read(Nat.0) + Nat.15.read(Nat.3) = (Nat.640 + Nat.15).read(Nat.0 + Nat.3)
    Nat.0 + Nat.3 = Nat.3
    (Nat.640 + Nat.15).read(Nat.0 + Nat.3) = (Nat.640 + Nat.15).read(Nat.3)
    Nat.640 = Nat.64.read(Nat.0)
    Nat.15 = Nat.1.read(Nat.5)
    read_add_read(Nat.64, Nat.0, Nat.1, Nat.5)
    Nat.64.read(Nat.0) + Nat.1.read(Nat.5) = (Nat.64 + Nat.1).read(Nat.0 + Nat.5)
    Nat.0 + Nat.5 = Nat.5
    (Nat.64 + Nat.1).read(Nat.0 + Nat.5) = (Nat.64 + Nat.1).read(Nat.5)
    Nat.64 + Nat.1 = Nat.65
    (Nat.64 + Nat.1).read(Nat.5) = Nat.65.read(Nat.5)
    Nat.65.read(Nat.5) = Nat.655
    Nat.640 + Nat.15 = Nat.655
    (Nat.640 + Nat.15).read(Nat.3) = Nat.655.read(Nat.3)
    Nat.655.read(Nat.3) = Nat.6553
    Nat.6400 + Nat.153 = Nat.6553
}

/// `256 * 256 = 65536`.
theorem nat_mul_256_256 {
    Nat.256 * Nat.256 = Nat.65536
} by {
    Nat.256 = Nat.25.read(Nat.6)
    read_mul_single(Nat.25, Nat.6, Nat.256)
    Nat.25.read(Nat.6) * Nat.256 = (Nat.25 * Nat.256).read(Nat.6 * Nat.256)
    nat_mul_25_256
    Nat.25 * Nat.256 = Nat.6400
    nat_mul_6_256
    Nat.6 * Nat.256 = Nat.1536
    Nat.6400.read(Nat.1536) = Nat.6400.read(Nat.10 * Nat.153 + Nat.6)
    read_read_carry(Nat.6400, Nat.153, Nat.6)
    Nat.6400.read(Nat.10 * Nat.153 + Nat.6) = (Nat.6400 + Nat.153).read(Nat.6)
    nat_add_6400_153
    Nat.6400 + Nat.153 = Nat.6553
    (Nat.6400 + Nat.153).read(Nat.6) = Nat.6553.read(Nat.6)
    Nat.6553.read(Nat.6) = Nat.65536
    Nat.256 * Nat.256 = Nat.65536
}

/// `2^8 = 256`, reached by doubling from `2^5 = 32`.
theorem two_pow_8_value {
    Nat.2.pow(Nat.8) = Nat.256
} by {
    exp_add(Nat.2, Nat.7, Nat.1)
    Nat.2.pow(Nat.7 + Nat.1) = Nat.2.pow(Nat.7) * Nat.2.pow(Nat.1)
    exp_one(Nat.2)
    Nat.2.pow(Nat.1) = Nat.2
    Nat.7 + Nat.1 = Nat.8
    Nat.2.pow(Nat.8) = Nat.2.pow(Nat.7) * Nat.2
    mul_comm(Nat.2.pow(Nat.7), Nat.2)
    Nat.2.pow(Nat.8) = Nat.2 * Nat.2.pow(Nat.7)
    exp_add(Nat.2, Nat.6, Nat.1)
    Nat.2.pow(Nat.6 + Nat.1) = Nat.2.pow(Nat.6) * Nat.2.pow(Nat.1)
    Nat.6 + Nat.1 = Nat.7
    Nat.2.pow(Nat.7) = Nat.2.pow(Nat.6) * Nat.2
    mul_comm(Nat.2.pow(Nat.6), Nat.2)
    Nat.2.pow(Nat.7) = Nat.2 * Nat.2.pow(Nat.6)
    exp_add(Nat.2, Nat.5, Nat.1)
    Nat.2.pow(Nat.5 + Nat.1) = Nat.2.pow(Nat.5) * Nat.2.pow(Nat.1)
    Nat.5 + Nat.1 = Nat.6
    Nat.2.pow(Nat.6) = Nat.2.pow(Nat.5) * Nat.2
    mul_comm(Nat.2.pow(Nat.5), Nat.2)
    Nat.2.pow(Nat.6) = Nat.2 * Nat.2.pow(Nat.5)
    pow_2_5_value
    Nat.2.pow(Nat.5) = Nat.32
    nat_mul_2_32
    Nat.2 * Nat.32 = Nat.64
    Nat.2.pow(Nat.6) = Nat.64
    nat_mul_2_64
    Nat.2 * Nat.64 = Nat.128
    Nat.2.pow(Nat.7) = Nat.128
    nat_mul_2_128
    Nat.2 * Nat.128 = Nat.256
    Nat.2.pow(Nat.8) = Nat.256
}

/// `2^16 = 65536`, the square of `2^8`.
theorem two_pow_16_value {
    Nat.2.pow(Nat.16) = Nat.65536
} by {
    exp_add(Nat.2, Nat.8, Nat.8)
    Nat.2.pow(Nat.8 + Nat.8) = Nat.2.pow(Nat.8) * Nat.2.pow(Nat.8)
    Nat.8 + Nat.8 = Nat.16
    Nat.2.pow(Nat.16) = Nat.2.pow(Nat.8) * Nat.2.pow(Nat.8)
    two_pow_8_value
    Nat.2.pow(Nat.8) = Nat.256
    Nat.2.pow(Nat.8) * Nat.2.pow(Nat.8) = Nat.256 * Nat.256
    nat_mul_256_256
    Nat.256 * Nat.256 = Nat.65536
    Nat.2.pow(Nat.16) = Nat.65536
}

// ---------------------------------------------------------------------------
// Target (a): the tower 3 ↑↑ 2 = 27 and 3 ↑↑ 3 = 3^27.
// ---------------------------------------------------------------------------

/// `3 ↑↑ 1 = 3`: a tower of height one is just the base.
theorem tetration_three_one {
    tetration(Nat.3, Nat.1) = Nat.3
} by {
    tetration_suc(Nat.3, Nat.0)
    tetration(Nat.3, Nat.1) = Nat.3.pow(tetration(Nat.3, Nat.0))
    tetration_zero(Nat.3)
    tetration(Nat.3, Nat.0) = Nat.1
    Nat.3.pow(tetration(Nat.3, Nat.0)) = Nat.3.pow(Nat.1)
    exp_one(Nat.3)
    Nat.3.pow(Nat.1) = Nat.3
    tetration(Nat.3, Nat.1) = Nat.3
}

/// `3 ↑↑ 2 = 3^3 = 27`.
theorem tetration_three_two {
    tetration(Nat.3, Nat.2) = Nat.27
} by {
    tetration_suc(Nat.3, Nat.1)
    tetration(Nat.3, Nat.2) = Nat.3.pow(tetration(Nat.3, Nat.1))
    tetration_three_one
    tetration(Nat.3, Nat.1) = Nat.3
    Nat.3.pow(tetration(Nat.3, Nat.1)) = Nat.3.pow(Nat.3)
    pow_3_3_value
    Nat.3.pow(Nat.3) = Nat.27
    tetration(Nat.3, Nat.2) = Nat.27
}

/// `3 ↑↑ 3 = 3^(3^3) = 3^27`, in tower form.
theorem tetration_three_three_tower {
    tetration(Nat.3, Nat.3) = Nat.3.pow(Nat.3.pow(Nat.3))
} by {
    tetration_suc(Nat.3, Nat.2)
    tetration(Nat.3, Nat.3) = Nat.3.pow(tetration(Nat.3, Nat.2))
    tetration_suc(Nat.3, Nat.1)
    tetration(Nat.3, Nat.2) = Nat.3.pow(tetration(Nat.3, Nat.1))
    tetration_three_one
    tetration(Nat.3, Nat.1) = Nat.3
    Nat.3.pow(tetration(Nat.3, Nat.1)) = Nat.3.pow(Nat.3)
    tetration(Nat.3, Nat.2) = Nat.3.pow(Nat.3)
    Nat.3.pow(tetration(Nat.3, Nat.2)) = Nat.3.pow(Nat.3.pow(Nat.3))
    tetration(Nat.3, Nat.3) = Nat.3.pow(Nat.3.pow(Nat.3))
}

/// `3 ↑↑ 3 = 3^27`.
theorem tetration_three_three {
    tetration(Nat.3, Nat.3) = Nat.3.pow(Nat.27)
} by {
    tetration_suc(Nat.3, Nat.2)
    tetration(Nat.3, Nat.3) = Nat.3.pow(tetration(Nat.3, Nat.2))
    tetration_three_two
    tetration(Nat.3, Nat.2) = Nat.27
    Nat.3.pow(tetration(Nat.3, Nat.2)) = Nat.3.pow(Nat.27)
    tetration(Nat.3, Nat.3) = Nat.3.pow(Nat.27)
}

/// The general arrow agrees on the tower: `3 ↑^1 2 = 3 ↑↑ 2 = 27`.
theorem up_arrow_three_two {
    up_arrow(Nat.3, Nat.1, Nat.2) = Nat.27
} by {
    up_arrow_bridge_one(Nat.3, Nat.2)
    up_arrow(Nat.3, Nat.1, Nat.2) = tetration(Nat.3, Nat.2)
    tetration_three_two
    tetration(Nat.3, Nat.2) = Nat.27
    up_arrow(Nat.3, Nat.1, Nat.2) = Nat.27
}

// ---------------------------------------------------------------------------
// Target (b): 2 ↑↑ 3 = 16 and 2 ↑↑ 4 = 65536.
// ---------------------------------------------------------------------------

/// `2 ↑↑ 1 = 2`.
theorem tetration_two_one {
    tetration(Nat.2, Nat.1) = Nat.2
} by {
    tetration_suc(Nat.2, Nat.0)
    tetration(Nat.2, Nat.1) = Nat.2.pow(tetration(Nat.2, Nat.0))
    tetration_zero(Nat.2)
    tetration(Nat.2, Nat.0) = Nat.1
    Nat.2.pow(tetration(Nat.2, Nat.0)) = Nat.2.pow(Nat.1)
    exp_one(Nat.2)
    Nat.2.pow(Nat.1) = Nat.2
    tetration(Nat.2, Nat.1) = Nat.2
}

/// `2 ↑↑ 2 = 2^2 = 4`.
theorem tetration_two_two {
    tetration(Nat.2, Nat.2) = Nat.4
} by {
    tetration_suc(Nat.2, Nat.1)
    tetration(Nat.2, Nat.2) = Nat.2.pow(tetration(Nat.2, Nat.1))
    tetration_two_one
    tetration(Nat.2, Nat.1) = Nat.2
    Nat.2.pow(tetration(Nat.2, Nat.1)) = Nat.2.pow(Nat.2)
    pow_2_2_value
    Nat.2.pow(Nat.2) = Nat.4
    tetration(Nat.2, Nat.2) = Nat.4
}

/// `2 ↑↑ 3 = 2^(2^2) = 16`.
theorem tetration_two_three {
    tetration(Nat.2, Nat.3) = Nat.16
} by {
    tetration_suc(Nat.2, Nat.2)
    tetration(Nat.2, Nat.3) = Nat.2.pow(tetration(Nat.2, Nat.2))
    tetration_two_two
    tetration(Nat.2, Nat.2) = Nat.4
    Nat.2.pow(tetration(Nat.2, Nat.2)) = Nat.2.pow(Nat.4)
    pow_2_4_value
    Nat.2.pow(Nat.4) = Nat.16
    tetration(Nat.2, Nat.3) = Nat.16
}

/// `2 ↑↑ 4 = 2^(2^(2^2))`, in tower form.
theorem tetration_two_four_tower {
    tetration(Nat.2, Nat.4) = Nat.2.pow(Nat.2.pow(Nat.2.pow(Nat.2)))
} by {
    tetration_suc(Nat.2, Nat.3)
    tetration(Nat.2, Nat.4) = Nat.2.pow(tetration(Nat.2, Nat.3))
    tetration_suc(Nat.2, Nat.2)
    tetration(Nat.2, Nat.3) = Nat.2.pow(tetration(Nat.2, Nat.2))
    tetration_suc(Nat.2, Nat.1)
    tetration(Nat.2, Nat.2) = Nat.2.pow(tetration(Nat.2, Nat.1))
    tetration_two_one
    tetration(Nat.2, Nat.1) = Nat.2
    Nat.2.pow(tetration(Nat.2, Nat.1)) = Nat.2.pow(Nat.2)
    tetration(Nat.2, Nat.2) = Nat.2.pow(Nat.2)
    Nat.2.pow(tetration(Nat.2, Nat.2)) = Nat.2.pow(Nat.2.pow(Nat.2))
    tetration(Nat.2, Nat.3) = Nat.2.pow(Nat.2.pow(Nat.2))
    Nat.2.pow(tetration(Nat.2, Nat.3)) = Nat.2.pow(Nat.2.pow(Nat.2.pow(Nat.2)))
    tetration(Nat.2, Nat.4) = Nat.2.pow(Nat.2.pow(Nat.2.pow(Nat.2)))
}

/// `2 ↑↑ 4 = 2^16 = 65536`.
theorem tetration_two_four {
    tetration(Nat.2, Nat.4) = Nat.65536
} by {
    tetration_suc(Nat.2, Nat.3)
    tetration(Nat.2, Nat.4) = Nat.2.pow(tetration(Nat.2, Nat.3))
    tetration_two_three
    tetration(Nat.2, Nat.3) = Nat.16
    Nat.2.pow(tetration(Nat.2, Nat.3)) = Nat.2.pow(Nat.16)
    two_pow_16_value
    Nat.2.pow(Nat.16) = Nat.65536
    tetration(Nat.2, Nat.4) = Nat.65536
}

/// The general arrow agrees: `2 ↑^1 4 = 2 ↑↑ 4 = 65536`.
theorem up_arrow_two_four {
    up_arrow(Nat.2, Nat.1, Nat.4) = Nat.65536
} by {
    up_arrow_bridge_one(Nat.2, Nat.4)
    up_arrow(Nat.2, Nat.1, Nat.4) = tetration(Nat.2, Nat.4)
    tetration_two_four
    tetration(Nat.2, Nat.4) = Nat.65536
    up_arrow(Nat.2, Nat.1, Nat.4) = Nat.65536
}

// ---------------------------------------------------------------------------
// Target (c): 3 ↑↑↑ 2 = 3 ↑↑ 3 = 3^27.
// ---------------------------------------------------------------------------

/// `3 ↑↑↑ 1 = 3`: one iteration of tetration is the base.
theorem up_arrow3_three_one {
    up_arrow3(Nat.3, Nat.1) = Nat.3
} by {
    up_arrow3_suc(Nat.3, Nat.0)
    up_arrow3(Nat.3, Nat.1) = tetration(Nat.3, up_arrow3(Nat.3, Nat.0))
    up_arrow3_zero(Nat.3)
    up_arrow3(Nat.3, Nat.0) = Nat.1
    tetration(Nat.3, up_arrow3(Nat.3, Nat.0)) = tetration(Nat.3, Nat.1)
    tetration_three_one
    tetration(Nat.3, Nat.1) = Nat.3
    up_arrow3(Nat.3, Nat.1) = Nat.3
}

/// `3 ↑↑↑ 2 = 3 ↑↑ 3`: the recursion `a ↑↑↑ 2 = a ↑↑ a` at `a = 3`.
theorem up_arrow3_three_two {
    up_arrow3(Nat.3, Nat.2) = tetration(Nat.3, Nat.3)
} by {
    up_arrow3_suc(Nat.3, Nat.1)
    up_arrow3(Nat.3, Nat.2) = tetration(Nat.3, up_arrow3(Nat.3, Nat.1))
    up_arrow3_three_one
    up_arrow3(Nat.3, Nat.1) = Nat.3
    tetration(Nat.3, up_arrow3(Nat.3, Nat.1)) = tetration(Nat.3, Nat.3)
    up_arrow3(Nat.3, Nat.2) = tetration(Nat.3, Nat.3)
}

/// `3 ↑↑↑ 2 = 3^27`.
theorem up_arrow3_three_two_value {
    up_arrow3(Nat.3, Nat.2) = Nat.3.pow(Nat.27)
} by {
    up_arrow3_three_two
    up_arrow3(Nat.3, Nat.2) = tetration(Nat.3, Nat.3)
    tetration_three_three
    tetration(Nat.3, Nat.3) = Nat.3.pow(Nat.27)
    up_arrow3(Nat.3, Nat.2) = Nat.3.pow(Nat.27)
}

/// The general recursion law at the double-arrow level:
/// `a ↑↑↑ 2 = a ↑↑ a` for every base a.
theorem up_arrow3_two_law(a: Nat) {
    up_arrow3(a, Nat.2) = tetration(a, a)
} by {
    up_arrow3_suc(a, Nat.1)
    up_arrow3(a, Nat.2) = tetration(a, up_arrow3(a, Nat.1))
    up_arrow3_suc(a, Nat.0)
    up_arrow3(a, Nat.1) = tetration(a, up_arrow3(a, Nat.0))
    up_arrow3_zero(a)
    up_arrow3(a, Nat.0) = Nat.1
    tetration(a, up_arrow3(a, Nat.0)) = tetration(a, Nat.1)
    tetration_suc(a, Nat.0)
    tetration(a, Nat.1) = a.pow(tetration(a, Nat.0))
    tetration_zero(a)
    tetration(a, Nat.0) = Nat.1
    a.pow(tetration(a, Nat.0)) = a.pow(Nat.1)
    exp_one(a)
    a.pow(Nat.1) = a
    tetration(a, Nat.1) = a
    up_arrow3(a, Nat.1) = a
    tetration(a, up_arrow3(a, Nat.1)) = tetration(a, a)
    up_arrow3(a, Nat.2) = tetration(a, a)
}

/// The general arrow at level two agrees: `3 ↑^2 2 = 3 ↑↑↑ 2 = 3 ↑↑ 3`.
theorem up_arrow_three_triple_two {
    up_arrow(Nat.3, Nat.2, Nat.2) = tetration(Nat.3, Nat.3)
} by {
    up_arrow_bridge_two(Nat.3, Nat.2)
    up_arrow(Nat.3, Nat.2, Nat.2) = up_arrow3(Nat.3, Nat.2)
    up_arrow3_three_two
    up_arrow3(Nat.3, Nat.2) = tetration(Nat.3, Nat.3)
    up_arrow(Nat.3, Nat.2, Nat.2) = tetration(Nat.3, Nat.3)
}

// ---------------------------------------------------------------------------
// Target (d): the growth of 2 ↑↑ n.
// ---------------------------------------------------------------------------

/// `2 ↑↑ 5 = 2^65536`: the next tower value is a power of 65536.
///
/// The decimal expansion of 2^65536 has 19729 digits; it is stated here as a
/// power rather than written out.  This is the sense in which `2 ↑↑ n` grows
/// astronomically fast: each step exponentiates the previous value.
theorem tetration_two_five {
    tetration(Nat.2, Nat.5) = Nat.2.pow(Nat.65536)
} by {
    tetration_suc(Nat.2, Nat.4)
    tetration(Nat.2, Nat.5) = Nat.2.pow(tetration(Nat.2, Nat.4))
    tetration_two_four
    tetration(Nat.2, Nat.4) = Nat.65536
    Nat.2.pow(tetration(Nat.2, Nat.4)) = Nat.2.pow(Nat.65536)
    tetration(Nat.2, Nat.5) = Nat.2.pow(Nat.65536)
}

// ---------------------------------------------------------------------------
// Target (e): Graham's number.
// ---------------------------------------------------------------------------

/// The Graham sequence: `graham_g(0) = g_1 = 3 ↑↑↑↑ 3`, and
/// `graham_g(k.suc) = g_{k+2} = 3 ↑^{g_{k+1}} 3`, so `graham_g(k) = g_{k+1}`.
///
/// Level four of the up-arrow (`3 ↑↑↑↑ 3`) is the quadruple arrow: an
/// iterated tower whose height is itself a tower — already beyond any
/// practical computation.
define graham_g(k: Nat) -> Nat {
    match k {
        Nat.zero {
            up_arrow(Nat.3, Nat.4, Nat.3)
        }
        Nat.suc(m) {
            up_arrow(Nat.3, graham_g(m), Nat.3)
        }
    }
}

/// `g_1 = 3 ↑↑↑↑ 3`: the base of the Graham sequence.
theorem graham_g_base {
    graham_g(Nat.0) = up_arrow(Nat.3, Nat.4, Nat.3)
}

/// `g_{k+2} = 3 ↑^{g_{k+1}} 3`: each Graham term has the previous term as its
/// arrow count.
theorem graham_g_suc(k: Nat) {
    graham_g(k.suc) = up_arrow(Nat.3, graham_g(k), Nat.3)
}

/// Graham's number: `G = g_64`, the sixty-fourth term of the Graham sequence.
///
/// Graham's number is the famous upper bound in Ramsey theory: it is the
/// number of arrows in the level of the up-arrow hierarchy needed for the
/// Graham–Rothschild theorem, so large that its decimal expansion is
/// unknowable in practice.  It is defined here (a finite natural number, well
/// within the scope of the definition) but its value is not computable: the
/// arrow count of each term exceeds every term before it.
let graham_number: Nat = graham_g(Nat.63)

/// Graham's number is the sixty-fourth term: `G = g_64`.
theorem graham_number_eq {
    graham_number = graham_g(Nat.63)
}
