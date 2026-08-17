from nat import Nat, from_nat, from_nat_zero, from_nat_one, from_nat_mul, exp_ne_zero,
    pos_of_ne_zero, lt_imp_lte_suc, lte_ref, lt_trans, lt_suc, lt_and_lte,
    pow_one, mul_to_zero, only_zero_lte_zero
from real import Real, log_some_of_pos_exists,
    log_one, log_mul, log_monotone, from_nat_lte_mono, from_nat_real_pos_of_ne_zero,
    add_lte_add, lte_trans
from data.basic.witness import choose_witness_spec
from data.nat.nat_range_sum import range_sum, range_sum_zero, range_sum_suc, range_sum_congr
from data.nat.nat_range_prod import range_prod, range_prod_zero, range_prod_suc, range_prod_positive
from data.nat.nat_prime_interval_product import prime_or_one, prime_or_one_prime,
    prime_or_one_composite
from data.nat.nat_primorial import primorial
from data.nat.nat_primorial_bound import primorial_lte_four_pow
from number_theory.von_mangoldt import is_prime_power, is_prime_power_base_of, prime_power_base,
    von_mangoldt, is_prime_power_of_prime_power, prime_power_base_of_prime_power,
    log_value_mul_nat, log_value_pow_nat

numerals Nat
numerals Real

/// The summand of Chebyshev's theta function: the logarithm of `n` when `n` is prime, and zero
/// otherwise.
///
/// The same device as the primorial: a sum over the whole range stands in for a sum over the
/// primes in it, with composite positions contributing nothing.
define chebyshev_theta_weight(n: Nat) -> Real {
    if n.is_prime {
        (from_nat[Real](n)).log.get_or_else(Real.0)
    } else {
        Real.0
    }
}

/// Chebyshev's theta function: the sum of `log p` over the primes `p <= x`.
define chebyshev_theta(x: Nat) -> Real {
    range_sum(chebyshev_theta_weight, x.suc)
}

/// The summand of Chebyshev's psi function: the logarithm of the prime base of `n` when `n` is
/// a prime power, and zero otherwise.
define chebyshev_psi_weight(n: Nat) -> Real {
    if is_prime_power(n) {
        (from_nat[Real](prime_power_base(n))).log.get_or_else(Real.0)
    } else {
        Real.0
    }
}

/// Chebyshev's psi function: the sum of `log p` over the prime powers `p^k <= x`.
define chebyshev_psi(x: Nat) -> Real {
    range_sum(chebyshev_psi_weight, x.suc)
}

/// The logarithm of a natural-valued summand, as a real.
define nat_log_weight(f: Nat -> Nat, i: Nat) -> Real {
    (from_nat[Real](f(i))).log.get_or_else(Real.0)
}

/// The logarithm of one is zero.
theorem log_value_one {
    (Real.1).log.get_or_else(Real.0) = Real.0
} by {
    log_one
    Real.1.log = Option.some(Real.0)
    option_get_or_else_some[Real](Real.0, Real.0)
    option_get_or_else(Option.some(Real.0), Real.0) = Real.0
    (Real.1).log.get_or_else(Real.0) = Real.0
}

/// The logarithm of a nonzero natural number is nonnegative.
theorem log_value_nonneg_nat(m: Nat) {
    m != Nat.0 implies Real.0 <= (from_nat[Real](m)).log.get_or_else(Real.0)
} by {
    if m != Nat.0 {
        pos_of_ne_zero(m)
        Nat.0 < m
        lt_imp_lte_suc(Nat.0, m)
        Nat.1 <= m
        from_nat_lte_mono(Nat.1, m)
        from_nat[Real](Nat.1) <= from_nat[Real](m)
        from_nat_one[Real]
        from_nat[Real](Nat.1) = Real.1
        Real.1 <= from_nat[Real](m)
        from_nat_real_pos_of_ne_zero(m)
        from_nat[Real](m) > Real.0
        Real.1 > Real.0
        log_some_of_pos_exists(Real.1)
        let l1: Real satisfy {
            Real.1.log = Option.some(l1)
        }
        option_get_or_else_some[Real](l1, Real.0)
        option_get_or_else(Option.some(l1), Real.0) = l1
        (Real.1).log.get_or_else(Real.0) = l1
        Real.1.log = Option.some((Real.1).log.get_or_else(Real.0))
        log_some_of_pos_exists(from_nat[Real](m))
        let lm: Real satisfy {
            (from_nat[Real](m)).log = Option.some(lm)
        }
        option_get_or_else_some[Real](lm, Real.0)
        option_get_or_else(Option.some(lm), Real.0) = lm
        (from_nat[Real](m)).log.get_or_else(Real.0) = lm
        (from_nat[Real](m)).log = Option.some((from_nat[Real](m)).log.get_or_else(Real.0))
        log_monotone(Real.1, from_nat[Real](m), (Real.1).log.get_or_else(Real.0),
            (from_nat[Real](m)).log.get_or_else(Real.0))
        (Real.1).log.get_or_else(Real.0) <= (from_nat[Real](m)).log.get_or_else(Real.0)
        log_value_one
        (Real.1).log.get_or_else(Real.0) = Real.0
        Real.0 <= (from_nat[Real](m)).log.get_or_else(Real.0)
    }
}

/// The primorial factor at every position is nonzero.
theorem prime_or_one_ne_zero(i: Nat) {
    prime_or_one(i) != Nat.0
} by {
    if i.is_prime {
        prime_or_one_prime(i)
        prime_or_one(i) = i
        Nat.1 < i
        i != Nat.0
        prime_or_one(i) != Nat.0
    }
    if not i.is_prime {
        prime_or_one_composite(i)
        prime_or_one(i) = Nat.1
        Nat.1 != Nat.0
        prime_or_one(i) != Nat.0
    }
    prime_or_one(i) != Nat.0
}

/// The primorial of `n` is positive.
theorem primorial_pos(n: Nat) {
    Nat.0 < primorial(n)
} by {
    primorial(n) = range_prod(prime_or_one, n.suc)
    forall(i: Nat) {
        if i < n.suc {
            prime_or_one_ne_zero(i)
            prime_or_one(i) != Nat.0
            pos_of_ne_zero(prime_or_one(i))
            Nat.0 < prime_or_one(i)
        }
        (i < n.suc implies Nat.0 < prime_or_one(i))
    }
    range_prod_positive(prime_or_one, n.suc)
    Nat.0 < range_prod(prime_or_one, n.suc)
    Nat.0 < primorial(n)
}

/// The logarithm of a range product of nonzero naturals is the range sum of the logarithms.
theorem log_value_range_prod_nat(f: Nat -> Nat, n: Nat) {
    (forall(i: Nat) { i < n implies f(i) != Nat.0 })
        implies (from_nat[Real](range_prod(f, n))).log.get_or_else(Real.0) =
            range_sum(nat_log_weight(f), n)
} by {
    define p(x: Nat) -> Bool {
        (forall(i: Nat) { i < x implies f(i) != Nat.0 })
            implies (from_nat[Real](range_prod(f, x))).log.get_or_else(Real.0) =
                range_sum(nat_log_weight(f), x)
    }
    range_prod_zero(f)
    range_prod(f, Nat.0) = Nat.1
    from_nat[Real](range_prod(f, Nat.0)) = from_nat[Real](Nat.1)
    from_nat_one[Real]
    from_nat[Real](Nat.1) = Real.1
    from_nat[Real](range_prod(f, Nat.0)) = Real.1
    (from_nat[Real](range_prod(f, Nat.0))).log.get_or_else(Real.0) = (Real.1).log.get_or_else(Real.0)
    log_value_one
    (Real.1).log.get_or_else(Real.0) = Real.0
    (from_nat[Real](range_prod(f, Nat.0))).log.get_or_else(Real.0) = Real.0
    range_sum_zero(nat_log_weight(f))
    range_sum(nat_log_weight(f), Nat.0) = Real.0
    (from_nat[Real](range_prod(f, Nat.0))).log.get_or_else(Real.0) =
        range_sum(nat_log_weight(f), Nat.0)
    p(Nat.0)
    forall(k: Nat) {
        if p(k) {
            if forall(i: Nat) { i < k.suc implies f(i) != Nat.0 } {
                forall(i: Nat) {
                    if i < k {
                        lt_suc(k)
                        k < k.suc
                        lt_trans(i, k, k.suc)
                        i < k.suc
                        f(i) != Nat.0
                    }
                    (i < k implies f(i) != Nat.0)
                }
                lt_suc(k)
                k < k.suc
                f(k) != Nat.0
                forall(i: Nat) {
                    if i < k {
                        f(i) != Nat.0
                        pos_of_ne_zero(f(i))
                        Nat.0 < f(i)
                    }
                    (i < k implies Nat.0 < f(i))
                }
                range_prod_positive(f, k)
                Nat.0 < range_prod(f, k)
                range_prod(f, k) != Nat.0
                range_prod_suc(f, k)
                range_prod(f, k.suc) = range_prod(f, k) * f(k)
                from_nat_mul[Real](range_prod(f, k), f(k))
                from_nat[Real](range_prod(f, k) * f(k)) =
                    from_nat[Real](range_prod(f, k)) * from_nat[Real](f(k))
                from_nat[Real](range_prod(f, k.suc)) =
                    from_nat[Real](range_prod(f, k)) * from_nat[Real](f(k))
                log_value_mul_nat(range_prod(f, k), f(k))
                (from_nat[Real](range_prod(f, k) * f(k))).log.get_or_else(Real.0) =
                    (from_nat[Real](range_prod(f, k))).log.get_or_else(Real.0) +
                        (from_nat[Real](f(k))).log.get_or_else(Real.0)
                (from_nat[Real](range_prod(f, k.suc))).log.get_or_else(Real.0) =
                    (from_nat[Real](range_prod(f, k))).log.get_or_else(Real.0) +
                        (from_nat[Real](f(k))).log.get_or_else(Real.0)
                p(k) = ((forall(i: Nat) { i < k implies f(i) != Nat.0 })
                    implies (from_nat[Real](range_prod(f, k))).log.get_or_else(Real.0) =
                        range_sum(nat_log_weight(f), k))
                p(k)
                (from_nat[Real](range_prod(f, k))).log.get_or_else(Real.0) =
                    range_sum(nat_log_weight(f), k)
                (from_nat[Real](range_prod(f, k.suc))).log.get_or_else(Real.0) =
                    range_sum(nat_log_weight(f), k) +
                        (from_nat[Real](f(k))).log.get_or_else(Real.0)
                nat_log_weight(f, k) = (from_nat[Real](f(k))).log.get_or_else(Real.0)
                (from_nat[Real](range_prod(f, k.suc))).log.get_or_else(Real.0) =
                    range_sum(nat_log_weight(f), k) + nat_log_weight(f, k)
                range_sum_suc(nat_log_weight(f), k)
                range_sum(nat_log_weight(f), k.suc) =
                    range_sum(nat_log_weight(f), k) + nat_log_weight(f, k)
                (from_nat[Real](range_prod(f, k.suc))).log.get_or_else(Real.0) =
                    range_sum(nat_log_weight(f), k.suc)
            }
            p(k.suc)
        }
        (p(k) implies p(k.suc))
    }
    p(Nat.0) and forall(k: Nat) {
        p(k) implies p(k.suc)
    }
    Nat.induction(p)
    p(n)
    if forall(i: Nat) { i < n implies f(i) != Nat.0 } {
        p(n) = ((forall(i: Nat) { i < n implies f(i) != Nat.0 })
            implies (from_nat[Real](range_prod(f, n))).log.get_or_else(Real.0) =
                range_sum(nat_log_weight(f), n))
        p(n)
        (from_nat[Real](range_prod(f, n))).log.get_or_else(Real.0) =
            range_sum(nat_log_weight(f), n)
    }
}

/// At every position the theta summand is the logarithm of the primorial factor.
theorem chebyshev_theta_weight_eq_log_value_prime_or_one(i: Nat) {
    chebyshev_theta_weight(i) = (from_nat[Real](prime_or_one(i))).log.get_or_else(Real.0)
} by {
    if i.is_prime {
        chebyshev_theta_weight(i) = if i.is_prime {
            (from_nat[Real](i)).log.get_or_else(Real.0)
        } else {
            Real.0
        }
        chebyshev_theta_weight(i) = (from_nat[Real](i)).log.get_or_else(Real.0)
        prime_or_one_prime(i)
        prime_or_one(i) = i
        (from_nat[Real](prime_or_one(i))).log.get_or_else(Real.0) = (from_nat[Real](i)).log.get_or_else(Real.0)
        chebyshev_theta_weight(i) = (from_nat[Real](prime_or_one(i))).log.get_or_else(Real.0)
    }
    if not i.is_prime {
        chebyshev_theta_weight(i) = if i.is_prime {
            (from_nat[Real](i)).log.get_or_else(Real.0)
        } else {
            Real.0
        }
        chebyshev_theta_weight(i) = Real.0
        prime_or_one_composite(i)
        prime_or_one(i) = Nat.1
        from_nat[Real](prime_or_one(i)) = from_nat[Real](Nat.1)
        from_nat_one[Real]
        from_nat[Real](Nat.1) = Real.1
        from_nat[Real](prime_or_one(i)) = Real.1
        (from_nat[Real](prime_or_one(i))).log.get_or_else(Real.0) = (Real.1).log.get_or_else(Real.0)
        log_value_one
        (Real.1).log.get_or_else(Real.0) = Real.0
        (from_nat[Real](prime_or_one(i))).log.get_or_else(Real.0) = Real.0
        chebyshev_theta_weight(i) = (from_nat[Real](prime_or_one(i))).log.get_or_else(Real.0)
    }
    chebyshev_theta_weight(i) = (from_nat[Real](prime_or_one(i))).log.get_or_else(Real.0)
}

/// Chebyshev's theta function is the logarithm of the primorial.
theorem chebyshev_theta_eq_log_value_primorial(n: Nat) {
    chebyshev_theta(n) = (from_nat[Real](primorial(n))).log.get_or_else(Real.0)
} by {
    forall(i: Nat) {
        if i < n.suc {
            prime_or_one_ne_zero(i)
            prime_or_one(i) != Nat.0
        }
        (i < n.suc implies prime_or_one(i) != Nat.0)
    }
    log_value_range_prod_nat(prime_or_one, n.suc)
    (from_nat[Real](range_prod(prime_or_one, n.suc))).log.get_or_else(Real.0) =
        range_sum(nat_log_weight(prime_or_one), n.suc)
    primorial(n) = range_prod(prime_or_one, n.suc)
    from_nat[Real](primorial(n)) = from_nat[Real](range_prod(prime_or_one, n.suc))
    (from_nat[Real](primorial(n))).log.get_or_else(Real.0) =
        range_sum(nat_log_weight(prime_or_one), n.suc)
    forall(i: Nat) {
        chebyshev_theta_weight_eq_log_value_prime_or_one(i)
        chebyshev_theta_weight(i) = (from_nat[Real](prime_or_one(i))).log.get_or_else(Real.0)
        nat_log_weight(prime_or_one, i) = (from_nat[Real](prime_or_one(i))).log.get_or_else(Real.0)
        chebyshev_theta_weight(i) = nat_log_weight(prime_or_one, i)
        (i < n.suc implies chebyshev_theta_weight(i) = nat_log_weight(prime_or_one, i))
    }
    range_sum_congr(chebyshev_theta_weight, nat_log_weight(prime_or_one), n.suc)
    range_sum(chebyshev_theta_weight, n.suc) =
        range_sum(nat_log_weight(prime_or_one), n.suc)
    (from_nat[Real](primorial(n))).log.get_or_else(Real.0) =
        range_sum(chebyshev_theta_weight, n.suc)
    chebyshev_theta(n) = range_sum(chebyshev_theta_weight, n.suc)
    chebyshev_theta(n) = (from_nat[Real](primorial(n))).log.get_or_else(Real.0)
}

/// The logarithm is monotone on natural arguments.
theorem log_value_monotone_nat(m: Nat, n: Nat) {
    m <= n and m != Nat.0 implies
        (from_nat[Real](m)).log.get_or_else(Real.0) <= (from_nat[Real](n)).log.get_or_else(Real.0)
} by {
    if m <= n and m != Nat.0 {
        from_nat_lte_mono(m, n)
        from_nat[Real](m) <= from_nat[Real](n)
        from_nat_real_pos_of_ne_zero(m)
        from_nat[Real](m) > Real.0
        if n = Nat.0 {
            m <= Nat.0
            only_zero_lte_zero(m)
            m = Nat.0
            m != Nat.0
            false
        }
        n != Nat.0
        from_nat_real_pos_of_ne_zero(n)
        from_nat[Real](n) > Real.0
        log_some_of_pos_exists(from_nat[Real](m))
        let lm: Real satisfy {
            (from_nat[Real](m)).log = Option.some(lm)
        }
        option_get_or_else_some[Real](lm, Real.0)
        option_get_or_else(Option.some(lm), Real.0) = lm
        (from_nat[Real](m)).log.get_or_else(Real.0) = lm
        (from_nat[Real](m)).log = Option.some((from_nat[Real](m)).log.get_or_else(Real.0))
        log_some_of_pos_exists(from_nat[Real](n))
        let ln: Real satisfy {
            (from_nat[Real](n)).log = Option.some(ln)
        }
        option_get_or_else_some[Real](ln, Real.0)
        option_get_or_else(Option.some(ln), Real.0) = ln
        (from_nat[Real](n)).log.get_or_else(Real.0) = ln
        (from_nat[Real](n)).log = Option.some((from_nat[Real](n)).log.get_or_else(Real.0))
        log_monotone(from_nat[Real](m), from_nat[Real](n),
            (from_nat[Real](m)).log.get_or_else(Real.0), (from_nat[Real](n)).log.get_or_else(Real.0))
        (from_nat[Real](m)).log.get_or_else(Real.0) <= (from_nat[Real](n)).log.get_or_else(Real.0)
    }
}

/// Chebyshev's theta function is at most `n * log 4`: the primorial bound.
theorem chebyshev_theta_lte_n_mul_log_four(n: Nat) {
    chebyshev_theta(n) <= from_nat[Real](n) * (from_nat[Real](Nat.4)).log.get_or_else(Real.0)
} by {
    chebyshev_theta_eq_log_value_primorial(n)
    chebyshev_theta(n) = (from_nat[Real](primorial(n))).log.get_or_else(Real.0)
    primorial_lte_four_pow(n)
    primorial(n) <= Nat.4.pow(n)
    from_nat_lte_mono(primorial(n), Nat.4.pow(n))
    from_nat[Real](primorial(n)) <= from_nat[Real](Nat.4.pow(n))
    primorial_pos(n)
    Nat.0 < primorial(n)
    primorial(n) != Nat.0
    Nat.4 != Nat.0
    exp_ne_zero(Nat.4, n)
    Nat.4.pow(n) != Nat.0
    log_value_monotone_nat(primorial(n), Nat.4.pow(n))
    (from_nat[Real](primorial(n))).log.get_or_else(Real.0) <= (from_nat[Real](Nat.4.pow(n))).log.get_or_else(Real.0)
    log_value_pow_nat(Nat.4, n)
    (from_nat[Real](Nat.4.pow(n))).log.get_or_else(Real.0) =
        from_nat[Real](n) * (from_nat[Real](Nat.4)).log.get_or_else(Real.0)
    lte_trans((from_nat[Real](primorial(n))).log.get_or_else(Real.0),
        (from_nat[Real](Nat.4.pow(n))).log.get_or_else(Real.0),
        from_nat[Real](n) * (from_nat[Real](Nat.4)).log.get_or_else(Real.0))
    (from_nat[Real](primorial(n))).log.get_or_else(Real.0) <= from_nat[Real](n) * (from_nat[Real](Nat.4)).log.get_or_else(Real.0)
    chebyshev_theta(n) <= from_nat[Real](n) * (from_nat[Real](Nat.4)).log.get_or_else(Real.0)
}

/// The psi summand is the von Mangoldt function.
theorem chebyshev_psi_weight_eq_von_mangoldt(i: Nat) {
    chebyshev_psi_weight(i) = von_mangoldt(i)
} by {
    chebyshev_psi_weight(i) = if is_prime_power(i) {
        (from_nat[Real](prime_power_base(i))).log.get_or_else(Real.0)
    } else {
        Real.0
    }
    von_mangoldt(i) = if is_prime_power(i) {
        (from_nat[Real](prime_power_base(i))).log.get_or_else(Real.0)
    } else {
        Real.0
    }
    chebyshev_psi_weight(i) = von_mangoldt(i)
}

/// Chebyshev's psi function is the sum of the von Mangoldt function: `psi(x) = sum_{n <= x} Lambda(n)`.
theorem chebyshev_psi_eq_range_sum_von_mangoldt(x: Nat) {
    chebyshev_psi(x) = range_sum(von_mangoldt, x.suc)
} by {
    forall(i: Nat) {
        chebyshev_psi_weight_eq_von_mangoldt(i)
        chebyshev_psi_weight(i) = von_mangoldt(i)
        (i < x.suc implies chebyshev_psi_weight(i) = von_mangoldt(i))
    }
    range_sum_congr(chebyshev_psi_weight, von_mangoldt, x.suc)
    range_sum(chebyshev_psi_weight, x.suc) = range_sum(von_mangoldt, x.suc)
    chebyshev_psi(x) = range_sum(chebyshev_psi_weight, x.suc)
    chebyshev_psi(x) = range_sum(von_mangoldt, x.suc)
}

/// Range sums of reals are monotone in the summand.
theorem range_sum_lte_real(f: Nat -> Real, g: Nat -> Real, n: Nat) {
    (forall(i: Nat) { f(i) <= g(i) }) implies range_sum(f, n) <= range_sum(g, n)
} by {
    if forall(i: Nat) { f(i) <= g(i) } {
        define p(x: Nat) -> Bool {
            range_sum(f, x) <= range_sum(g, x)
        }
        range_sum_zero(f)
        range_sum_zero(g)
        range_sum(f, Nat.0) <= range_sum(g, Nat.0)
        p(Nat.0)
        forall(k: Nat) {
            if p(k) {
                range_sum(f, k) <= range_sum(g, k)
                f(k) <= g(k)
                add_lte_add(range_sum(f, k), range_sum(g, k), f(k), g(k))
                range_sum(f, k) + f(k) <= range_sum(g, k) + g(k)
                range_sum_suc(f, k)
                range_sum_suc(g, k)
                range_sum(f, k.suc) <= range_sum(g, k.suc)
                p(k.suc)
            }
            (p(k) implies p(k.suc))
        }
        p(Nat.0) and forall(k: Nat) {
            p(k) implies p(k.suc)
        }
        Nat.induction(p)
        p(n)
        range_sum(f, n) <= range_sum(g, n)
    }
}

/// The theta summand is at most the psi summand.
theorem chebyshev_theta_weight_lte_psi_weight(i: Nat) {
    chebyshev_theta_weight(i) <= chebyshev_psi_weight(i)
} by {
    if i.is_prime {
        chebyshev_theta_weight(i) = if i.is_prime {
            (from_nat[Real](i)).log.get_or_else(Real.0)
        } else {
            Real.0
        }
        chebyshev_theta_weight(i) = (from_nat[Real](i)).log.get_or_else(Real.0)
        is_prime_power_of_prime_power(i, Nat.1)
        is_prime_power(i.pow(Nat.1))
        pow_one(i)
        i.pow(Nat.1) = i
        is_prime_power(i)
        lte_ref(Nat.1)
        Nat.1 <= Nat.1
        prime_power_base_of_prime_power(i, Nat.1)
        prime_power_base(i.pow(Nat.1)) = i
        prime_power_base(i) = i
        chebyshev_psi_weight(i) = if is_prime_power(i) {
            (from_nat[Real](prime_power_base(i))).log.get_or_else(Real.0)
        } else {
            Real.0
        }
        chebyshev_psi_weight(i) = (from_nat[Real](prime_power_base(i))).log.get_or_else(Real.0)
        chebyshev_psi_weight(i) = (from_nat[Real](i)).log.get_or_else(Real.0)
        chebyshev_theta_weight(i) = chebyshev_psi_weight(i)
        chebyshev_theta_weight(i) <= chebyshev_psi_weight(i)
    }
    if not i.is_prime {
        chebyshev_theta_weight(i) = if i.is_prime {
            (from_nat[Real](i)).log.get_or_else(Real.0)
        } else {
            Real.0
        }
        chebyshev_theta_weight(i) = Real.0
        chebyshev_psi_weight(i) = if is_prime_power(i) {
            (from_nat[Real](prime_power_base(i))).log.get_or_else(Real.0)
        } else {
            Real.0
        }
        if is_prime_power(i) {
            is_prime_power(i) = exists(p: Nat) { is_prime_power_base_of(i)(p) }
            choose_witness_spec(is_prime_power_base_of(i))
            is_prime_power_base_of(i)(prime_power_base(i))
            is_prime_power_base_of(i)(prime_power_base(i)) =
                (prime_power_base(i).is_prime and exists(k: Nat) {
                    Nat.1 <= k and prime_power_base(i).pow(k) = i
                })
            prime_power_base(i).is_prime
            Nat.1 < prime_power_base(i)
            prime_power_base(i) != Nat.0
            chebyshev_psi_weight(i) = (from_nat[Real](prime_power_base(i))).log.get_or_else(Real.0)
            log_value_nonneg_nat(prime_power_base(i))
            Real.0 <= (from_nat[Real](prime_power_base(i))).log.get_or_else(Real.0)
            Real.0 <= chebyshev_psi_weight(i)
        }
        if not is_prime_power(i) {
            chebyshev_psi_weight(i) = Real.0
            Real.0 <= chebyshev_psi_weight(i)
        }
        Real.0 <= chebyshev_psi_weight(i)
        chebyshev_theta_weight(i) <= chebyshev_psi_weight(i)
    }
    chebyshev_theta_weight(i) <= chebyshev_psi_weight(i)
}

/// Chebyshev's theta function is at most Chebyshev's psi function.
theorem chebyshev_theta_lte_psi(x: Nat) {
    chebyshev_theta(x) <= chebyshev_psi(x)
} by {
    forall(i: Nat) {
        chebyshev_theta_weight_lte_psi_weight(i)
        chebyshev_theta_weight(i) <= chebyshev_psi_weight(i)
    }
    range_sum_lte_real(chebyshev_theta_weight, chebyshev_psi_weight, x.suc)
    range_sum(chebyshev_theta_weight, x.suc) <= range_sum(chebyshev_psi_weight, x.suc)
    chebyshev_theta(x) = range_sum(chebyshev_theta_weight, x.suc)
    chebyshev_psi(x) = range_sum(chebyshev_psi_weight, x.suc)
    chebyshev_theta(x) <= chebyshev_psi(x)
}

// The sharp comparison between the two Chebyshev functions,
//     theta(x) <= psi(x) <= theta(x) + O(sqrt(x) log x),
// is the entry to the prime number theorem.  The difference
//     psi(x) - theta(x) = sum_{p^k <= x, k >= 2} log p
// receives a contribution only from prime powers with exponent at least two.  A prime power
// `p^k <= x` with `k >= 2` has `p <= sqrt(x)`, and the powers of such a prime that lie below
// `x` number at most `log x / log p`, so
//     psi(x) - theta(x) <= sum_{p <= sqrt(x)} (log x / log p) * log p
//                          = pi(sqrt(x)) * log x <= sqrt(x) * log x.
// The statement needs the square root of a natural and a counting bound on the primes, so it is
// recorded here for future work rather than proved.
// theorem chebyshev_psi_lte_theta_add_sqrt_log(x: Nat) {
//     chebyshev_psi(x) <=
//         chebyshev_theta(x) + from_nat[Real](x.sqrt) * (from_nat[Real](x)).log.get_or_else(Real.0)
// }
