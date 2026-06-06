from nat import Nat, add_one_right, div_imp_mod, div_mod_decomp, succ_div_yes, succ_div_no, zero_div
from number_theory.legendre import prime_factor_count_upto, prime_factor_count_upto_step, prime_factor_count_upto_zero, prime_divides_iff_count_ne_zero, legendre_factorial
from number_theory.factorisation import count_prime_factor, count_prime_factor_mul, count_prime_factor_self
numerals Nat

/// A rearrangement used to fold the contribution of one extra multiple of `p`.
theorem add_shuffle(a: Nat, b: Nat, c: Nat) {
    a.suc + (b + c) = a + b + (c + Nat.1)
} by {
    a.suc + (b + c) = (a + b + c).suc
    a + b + (c + Nat.1) = (a + b + c).suc
}

/// Legendre's recurrence for the running valuation total `Σ_{k=1}^n v_p(k)`:
/// the multiples of `p` in `1, ..., n` contribute one each plus the valuations
/// of `1, ..., n div p`, while the non-multiples contribute nothing.
theorem legendre_recurrence(p: Nat, n: Nat) {
    p.is_prime implies
        prime_factor_count_upto(p, n) =
            n.div(p) + prime_factor_count_upto(p, n.div(p))
} by {
    if p.is_prime {
        p != Nat.0
        let f: Nat -> Bool = function(x: Nat) {
            prime_factor_count_upto(p, x) =
                x.div(p) + prime_factor_count_upto(p, x.div(p))
        }
        zero_div(p)
        prime_factor_count_upto_zero(p)
        f(Nat.0)
        forall(x: Nat) {
            if f(x) {
                let dd: Nat = x.div(p)
                prime_factor_count_upto(p, x) = dd + prime_factor_count_upto(p, dd)
                prime_factor_count_upto_step(p, x)
                prime_factor_count_upto_step(p, dd)
                add_one_right(dd)
                if p.divides(x.suc) {
                    succ_div_yes(x, p)
                    x.suc.div(p) = dd.suc
                    div_mod_decomp(x.suc, p)
                    div_imp_mod(x.suc, p)
                    x.suc.div(p) * p = x.suc
                    x.suc = dd.suc * p
                    dd.suc != Nat.0
                    count_prime_factor_mul(p, dd.suc, p)
                    count_prime_factor_self(p)
                    count_prime_factor(p, dd.suc * p) = count_prime_factor(p, dd.suc) + Nat.1
                    count_prime_factor(p, x.suc) = count_prime_factor(p, dd.suc) + Nat.1
                    prime_factor_count_upto(p, x.suc) =
                        dd + prime_factor_count_upto(p, dd) + (count_prime_factor(p, dd.suc) + Nat.1)
                    prime_factor_count_upto(p, dd.suc) =
                        prime_factor_count_upto(p, dd) + count_prime_factor(p, dd.suc)
                    add_shuffle(dd, prime_factor_count_upto(p, dd), count_prime_factor(p, dd.suc))
                    dd.suc + prime_factor_count_upto(p, dd.suc) =
                        dd + prime_factor_count_upto(p, dd) + (count_prime_factor(p, dd.suc) + Nat.1)
                    prime_factor_count_upto(p, x.suc) =
                        dd.suc + prime_factor_count_upto(p, dd.suc)
                    f(x.suc)
                }
                if not p.divides(x.suc) {
                    succ_div_no(x, p)
                    prime_divides_iff_count_ne_zero(p, x.suc)
                    count_prime_factor(p, x.suc) = Nat.0
                    prime_factor_count_upto(p, x.suc) =
                        dd + prime_factor_count_upto(p, dd)
                    f(x.suc)
                }
                f(x.suc)
            }
        }
        f(n)
        prime_factor_count_upto(p, n) =
            n.div(p) + prime_factor_count_upto(p, n.div(p))
    }
}

/// Legendre's recurrence stated directly for the valuation of `n!`.
theorem legendre_factorial_recurrence(p: Nat, n: Nat) {
    p.is_prime implies
        count_prime_factor(p, n.factorial) =
            n.div(p) + count_prime_factor(p, n.div(p).factorial)
} by {
    if p.is_prime {
        legendre_factorial(p, n)
        legendre_recurrence(p, n)
        legendre_factorial(p, n.div(p))
        count_prime_factor(p, n.factorial) =
            n.div(p) + count_prime_factor(p, n.div(p).factorial)
    }
}
