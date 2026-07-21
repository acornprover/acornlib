from number_theory.wilson import Nat, prime_pred_self_inverse_congr,
    prime_imp_wilson_factorial_congr
from number_theory.congruence import congr_mod_refl, congr_mod_symm,
    congr_mod_trans, congr_mod_mul, congr_mod_pow, mod_add_mul, mod_lt
from nat import exp_one, exp_add, exp_mul, one_exp, add_mod, small_mod,
    lt_add_suc, lt_suc_right, not_lt_zero, alt_suc_ne_zero,
    add_imp_sub_left, zero_or_suc, suc_sub_one, lt_suc, lte_mul, lt_and_lte,
    add_sub, divides_self, divides_mul, factorial_one, factorial_step,
    alt_induction, add_one_right, lte_and_lt, lte_mul_both, cross_sum_lte,
    lt_imp_lte_suc, mul_cancel_left, sum_lte, sq_eq_mul
from nat import divides_zero, sub_one_lt, sub_pos, mod_of_zero,
    lte_imp_not_lt, lt_not_ref, div_mod_decomp, div_imp_mod, add_imp_sub,
    distrib_left, distrib_right, mul_two_left, lt_diff, add_cancels_left,
    trichotomy, lte_trans, mul_assoc, lt_mul_both, lt_add_left, lt_trans,
    sub_lt, sub_self, sub_zero
from number_theory.falling_product import falling_product,
    falling_product_suc, falling_product_mul_factorial_complement
from number_theory.fermat import divides_imp_congr_zero,
    prime_does_not_divide_factorial
from number_theory.totient import congr_mod_add_cancel_right_pos,
    not_coprime_imp_divides_prime, coprime_below_prime
from number_theory.modular_inverse import cancel_coprime
from number_theory.primitive_root import is_order_double_unit_generator_mod,
    existing_order_double_unit_generator_euler_criterion_iff
from number_theory.arithmetic_functions import nat_mul_swap_middle
from list import List, map, product, is_permutation, map_add, map_singleton,
    product_append, map_range_locally_injective_bounded_is_permutation,
    permutation_preserves_mapped_product
from number_theory.quadratic_residue import is_quadratic_residue_mod,
    is_unit_quadratic_residue_mod, quadratic_residue_of_square_congr,
    quadratic_residue_coprime_is_unit, unit_quadratic_residue_is_residue,
    euler_criterion_quadratic_residue_forward
from number_theory.legendre_symbol import Int, legendre_symbol,
    is_quadratic_nonresidue_mod, quadratic_nonresidue_of_not_residue,
    legendre_symbol_one_of_nonzero_residue,
    legendre_symbol_neg_one_of_nonzero_nonresidue,
    legendre_symbol_value_one_iff_prime_unit_quadratic_residue,
    legendre_symbol_prime_coprime_unit_value_cases
numerals Nat
numerals Int

/// The positive-half representative of twice `x` modulo the odd number
/// `2h + 1`.
define doubling_signed_representative(h: Nat, x: Nat) -> Nat {
    if Nat.2 * x <= h {
        Nat.2 * x
    } else {
        Nat.2 * h + Nat.1 - Nat.2 * x
    }
}

/// The zero-based form of the signed doubling representative.
define doubling_signed_index(h: Nat) -> (Nat -> Nat) {
    function(i: Nat) {
        doubling_signed_representative(h, i.suc) - Nat.1
    }
}

/// The product of the positive signed representatives of the doubles from
/// one through `h`.
define doubling_signed_product(h: Nat) -> Nat {
    product[Nat](map(
        map(h.range, doubling_signed_index(h)), Nat.suc))
}

/// The signed doubling product restricted to the first `k` positive
/// integers.
define doubling_signed_product_prefix(h: Nat, k: Nat) -> Nat {
    product[Nat](map(
        map(k.range, doubling_signed_index(h)), Nat.suc))
}

/// Signed doubling carries every positive element at most `h` back into the
/// same positive half.
theorem doubling_signed_representative_positive_bounded(
    h: Nat, x: Nat
) {
    Nat.0 < x and x <= h implies
        Nat.0 < doubling_signed_representative(h, x) and
            doubling_signed_representative(h, x) <= h
} by {
    if Nat.0 < x and x <= h {
        if Nat.2 * x <= h {
            doubling_signed_representative(h, x) = Nat.2 * x
            alt_suc_ne_zero(Nat.1)
            Nat.2 != Nat.0
            lt_mul_both(Nat.2, Nat.0, x)
            Nat.2 * Nat.0 < Nat.2 * x
            Nat.2 * Nat.0 = Nat.0
            Nat.0 < Nat.2 * x
            Nat.0 < doubling_signed_representative(h, x)
            doubling_signed_representative(h, x) <= h
            Nat.0 < doubling_signed_representative(h, x) and
                doubling_signed_representative(h, x) <= h
        } else {
            h < Nat.2 * x
            alt_suc_ne_zero(Nat.1)
            Nat.2 != Nat.0
            lte_mul_both(Nat.2, x, h)
            Nat.2 * x <= Nat.2 * h
            lt_add_suc(Nat.2 * h, Nat.0)
            Nat.0.suc = Nat.1
            Nat.2 * h < Nat.2 * h + Nat.1
            lte_and_lt(Nat.2 * x, Nat.2 * h, Nat.2 * h + Nat.1)
            Nat.2 * x < Nat.2 * h + Nat.1
            doubling_signed_representative(h, x) =
                Nat.2 * h + Nat.1 - Nat.2 * x
            Nat.2 * h + Nat.1 - Nat.2 * x =
                doubling_signed_representative(h, x)
            sub_pos(Nat.2 * h + Nat.1, Nat.2 * x)
            Nat.0 < Nat.2 * h + Nat.1 - Nat.2 * x
            Nat.0 < doubling_signed_representative(h, x)
            add_sub(Nat.2 * h + Nat.1, Nat.2 * x)
            (Nat.2 * h + Nat.1 - Nat.2 * x) + Nat.2 * x =
                Nat.2 * h + Nat.1
            doubling_signed_representative(h, x) + Nat.2 * x =
                Nat.2 * h + Nat.1
            Nat.2 * x + doubling_signed_representative(h, x) =
                Nat.2 * h + Nat.1
            lt_imp_lte_suc(h, Nat.2 * x)
            h.suc <= Nat.2 * x
            add_one_right(h)
            h.suc = h + Nat.1
            Nat.2 * h + Nat.1 = (h + Nat.1) + h
            (h + Nat.1) + h =
                Nat.2 * x + doubling_signed_representative(h, x)
            cross_sum_lte(
                h + Nat.1, h, Nat.2 * x,
                doubling_signed_representative(h, x))
            doubling_signed_representative(h, x) <= h
            Nat.0 < doubling_signed_representative(h, x) and
                doubling_signed_representative(h, x) <= h
        }
    }
}

/// Signed doubling is injective on the positive half of an odd residue
/// system.
theorem doubling_signed_representative_injective(
    h: Nat, x: Nat, y: Nat
) {
    Nat.0 < x and x <= h and Nat.0 < y and y <= h and
        doubling_signed_representative(h, x) =
            doubling_signed_representative(h, y)
        implies x = y
} by {
    if Nat.0 < x and x <= h and Nat.0 < y and y <= h and
            doubling_signed_representative(h, x) =
                doubling_signed_representative(h, y) {
        lte_mul_both(Nat.2, x, h)
        Nat.2 * x <= Nat.2 * h
        lte_mul_both(Nat.2, y, h)
        Nat.2 * y <= Nat.2 * h
        sum_lte(Nat.2 * h, Nat.0, Nat.2 * h, Nat.1)
        Nat.2 * h + Nat.0 <= Nat.2 * h + Nat.1
        Nat.2 * h <= Nat.2 * h + Nat.1
        lte_trans(Nat.2 * x, Nat.2 * h, Nat.2 * h + Nat.1)
        Nat.2 * x <= Nat.2 * h + Nat.1
        lte_trans(Nat.2 * y, Nat.2 * h, Nat.2 * h + Nat.1)
        Nat.2 * y <= Nat.2 * h + Nat.1
        if Nat.2 * x <= h {
            doubling_signed_representative(h, x) = Nat.2 * x
            if Nat.2 * y <= h {
                doubling_signed_representative(h, y) = Nat.2 * y
                Nat.2 * x = Nat.2 * y
                alt_suc_ne_zero(Nat.1)
                Nat.2 != Nat.0
                mul_cancel_left(Nat.2, x, y)
                x = y
            } else {
                doubling_signed_representative(h, y) =
                    Nat.2 * h + Nat.1 - Nat.2 * y
                Nat.2 * x = Nat.2 * h + Nat.1 - Nat.2 * y
                Nat.2 * y <= Nat.2 * h + Nat.1
                add_sub(Nat.2 * h + Nat.1, Nat.2 * y)
                (Nat.2 * h + Nat.1 - Nat.2 * y) + Nat.2 * y =
                    Nat.2 * h + Nat.1
                Nat.2 * x + Nat.2 * y = Nat.2 * h + Nat.1
                Nat.2 * (x + y) = Nat.2 * h + Nat.1
                divides_self(Nat.2)
                Nat.2.divides(Nat.2)
                divides_mul(Nat.2, x + y, Nat.2)
                Nat.2.divides(Nat.2 * (x + y))
                Nat.2.divides(Nat.2 * h + Nat.1)
                div_imp_mod(Nat.2 * h + Nat.1, Nat.2)
                (Nat.2 * h + Nat.1).mod(Nat.2) = Nat.0
                Nat.2 * h = h * Nat.2
                mod_add_mul(h, Nat.2, Nat.1)
                (h * Nat.2 + Nat.1).mod(Nat.2) = Nat.1.mod(Nat.2)
                lt_suc(Nat.1)
                Nat.1.suc = Nat.2
                Nat.1 < Nat.2
                small_mod(Nat.1, Nat.2)
                Nat.1.mod(Nat.2) = Nat.1
                false
            }
            x = y
        } else {
            doubling_signed_representative(h, x) =
                Nat.2 * h + Nat.1 - Nat.2 * x
            if Nat.2 * y <= h {
                doubling_signed_representative(h, y) = Nat.2 * y
                Nat.2 * h + Nat.1 - Nat.2 * x = Nat.2 * y
                Nat.2 * x <= Nat.2 * h + Nat.1
                add_sub(Nat.2 * h + Nat.1, Nat.2 * x)
                (Nat.2 * h + Nat.1 - Nat.2 * x) + Nat.2 * x =
                    Nat.2 * h + Nat.1
                Nat.2 * y + Nat.2 * x = Nat.2 * h + Nat.1
                Nat.2 * (x + y) = Nat.2 * h + Nat.1
                divides_self(Nat.2)
                Nat.2.divides(Nat.2)
                divides_mul(Nat.2, x + y, Nat.2)
                Nat.2.divides(Nat.2 * (x + y))
                Nat.2.divides(Nat.2 * h + Nat.1)
                div_imp_mod(Nat.2 * h + Nat.1, Nat.2)
                (Nat.2 * h + Nat.1).mod(Nat.2) = Nat.0
                Nat.2 * h = h * Nat.2
                mod_add_mul(h, Nat.2, Nat.1)
                (h * Nat.2 + Nat.1).mod(Nat.2) = Nat.1.mod(Nat.2)
                lt_suc(Nat.1)
                Nat.1.suc = Nat.2
                Nat.1 < Nat.2
                small_mod(Nat.1, Nat.2)
                Nat.1.mod(Nat.2) = Nat.1
                false
            } else {
                doubling_signed_representative(h, y) =
                    Nat.2 * h + Nat.1 - Nat.2 * y
                Nat.2 * h + Nat.1 - Nat.2 * x =
                    Nat.2 * h + Nat.1 - Nat.2 * y
                Nat.2 * x <= Nat.2 * h + Nat.1
                Nat.2 * y <= Nat.2 * h + Nat.1
                add_sub(Nat.2 * h + Nat.1, Nat.2 * x)
                add_sub(Nat.2 * h + Nat.1, Nat.2 * y)
                (Nat.2 * h + Nat.1 - Nat.2 * x) + Nat.2 * x =
                    Nat.2 * h + Nat.1
                (Nat.2 * h + Nat.1 - Nat.2 * y) + Nat.2 * y =
                    Nat.2 * h + Nat.1
                (Nat.2 * h + Nat.1 - Nat.2 * x) + Nat.2 * x =
                    (Nat.2 * h + Nat.1 - Nat.2 * x) + Nat.2 * y
                add_cancels_left(
                    Nat.2 * h + Nat.1 - Nat.2 * x,
                    Nat.2 * x, Nat.2 * y)
                Nat.2 * x = Nat.2 * y
                alt_suc_ne_zero(Nat.1)
                Nat.2 != Nat.0
                mul_cancel_left(Nat.2, x, y)
                x = y
            }
            x = y
        }
        x = y
    }
}

/// The zero-based signed doubling index lies below the half-size.
theorem doubling_signed_index_bounded(h: Nat, i: Nat) {
    i < h implies doubling_signed_index(h)(i) < h
} by {
    if i < h {
        lt_imp_lte_suc(i, h)
        i.suc <= h
        alt_suc_ne_zero(i)
        i.suc != Nat.0
        Nat.0 < i.suc
        doubling_signed_representative_positive_bounded(h, i.suc)
        Nat.0 < doubling_signed_representative(h, i.suc)
        doubling_signed_representative(h, i.suc) <= h
        doubling_signed_index(h)(i) =
            doubling_signed_representative(h, i.suc) - Nat.1
        sub_one_lt(doubling_signed_representative(h, i.suc))
        doubling_signed_index(h)(i) < doubling_signed_representative(h, i.suc)
        lt_and_lte(
            doubling_signed_index(h)(i),
            doubling_signed_representative(h, i.suc), h)
        doubling_signed_index(h)(i) < h
    }
}

/// The zero-based signed doubling index is injective below the half-size.
theorem doubling_signed_index_injective_below(
    h: Nat, i: Nat, j: Nat
) {
    i < h and j < h and
        doubling_signed_index(h)(i) = doubling_signed_index(h)(j)
        implies i = j
} by {
    if i < h and j < h and
            doubling_signed_index(h)(i) = doubling_signed_index(h)(j) {
        lt_imp_lte_suc(i, h)
        i.suc <= h
        lt_imp_lte_suc(j, h)
        j.suc <= h
        alt_suc_ne_zero(i)
        i.suc != Nat.0
        Nat.0 < i.suc
        alt_suc_ne_zero(j)
        j.suc != Nat.0
        Nat.0 < j.suc
        doubling_signed_representative_positive_bounded(h, i.suc)
        Nat.0 < doubling_signed_representative(h, i.suc)
        doubling_signed_representative_positive_bounded(h, j.suc)
        Nat.0 < doubling_signed_representative(h, j.suc)
        doubling_signed_index(h)(i) =
            doubling_signed_representative(h, i.suc) - Nat.1
        doubling_signed_index(h)(j) =
            doubling_signed_representative(h, j.suc) - Nat.1
        doubling_signed_representative(h, i.suc) - Nat.1 =
            doubling_signed_representative(h, j.suc) - Nat.1
        add_sub(doubling_signed_representative(h, i.suc), Nat.1)
        add_sub(doubling_signed_representative(h, j.suc), Nat.1)
        (doubling_signed_representative(h, i.suc) - Nat.1) + Nat.1 =
            doubling_signed_representative(h, i.suc)
        (doubling_signed_representative(h, j.suc) - Nat.1) + Nat.1 =
            doubling_signed_representative(h, j.suc)
        doubling_signed_representative(h, i.suc) =
            doubling_signed_representative(h, j.suc)
        doubling_signed_representative_injective(h, i.suc, j.suc)
        i.suc = j.suc
        i = j
    }
}

/// The signed doubling indices permute all indices below the half-size.
theorem doubling_signed_indices_permute_range(h: Nat) {
    is_permutation(
        map(h.range, doubling_signed_index(h)), h.range)
} by {
    forall(i: Nat, j: Nat) {
        if i < h and j < h and
                doubling_signed_index(h)(i) = doubling_signed_index(h)(j) {
            doubling_signed_index_injective_below(h, i, j)
            i = j
        }
    }
    forall(i: Nat, j: Nat) {
        (i < h and j < h and doubling_signed_index(h)(i) =
            doubling_signed_index(h)(j)) implies i = j
    }
    forall(i: Nat) {
        if i < h {
            doubling_signed_index_bounded(h, i)
            doubling_signed_index(h)(i) < h
        }
    }
    forall(i: Nat) {
        i < h implies doubling_signed_index(h)(i) < h
    }
    let locally_injective = forall(i: Nat, j: Nat) {
        (i < h and j < h and doubling_signed_index(h)(i) =
            doubling_signed_index(h)(j)) implies i = j
    }
    let bounded = forall(i: Nat) {
        i < h implies doubling_signed_index(h)(i) < h
    }
    locally_injective
    bounded
    locally_injective and bounded
    map_range_locally_injective_bounded_is_permutation(
        h, doubling_signed_index(h))
    is_permutation(
        map(h.range, doubling_signed_index(h)), h.range)
}

/// The product of one plus every index below `h` is `h!`.
theorem product_succ_range_eq_factorial(h: Nat) {
    product[Nat](map(h.range, Nat.suc)) = h.factorial
} by {
    define pred(k: Nat) -> Bool {
        product[Nat](map(k.range, Nat.suc)) = k.factorial
    }
    Nat.0.range = List.nil[Nat]
    map(List.nil[Nat], Nat.suc) = List.nil[Nat]
    product[Nat](List.nil[Nat]) = Nat.1
    Nat.0.factorial = Nat.1
    pred(Nat.0)
    forall(k: Nat) {
        if pred(k) {
            product[Nat](map(k.range, Nat.suc)) = k.factorial
            k.suc.range = k.range.append(k)
            k.range.append(k) = k.range + List.singleton(k)
            map_add[Nat, Nat](k.range, List.singleton(k), Nat.suc)
            map_singleton[Nat, Nat](Nat.suc, k)
            map(k.suc.range, Nat.suc) =
                map(k.range, Nat.suc).append(k.suc)
            product_append[Nat](
                map(k.range, Nat.suc), List.singleton(k.suc))
            product[Nat](List.singleton(k.suc)) = k.suc
            product[Nat](map(k.suc.range, Nat.suc)) =
                product[Nat](map(k.range, Nat.suc)) * k.suc
            factorial_step(k)
            k.suc.factorial = k.suc * k.factorial
            product[Nat](map(k.range, Nat.suc)) * k.suc =
                k.factorial * k.suc
            k.factorial * k.suc = k.suc * k.factorial
            product[Nat](map(k.range, Nat.suc)) * k.suc =
                k.suc * k.factorial
            product[Nat](map(k.suc.range, Nat.suc)) = k.suc.factorial
            pred(k.suc)
        }
    }
    forall(k: Nat) { pred(k) implies pred(k.suc) }
    pred(Nat.0) and forall(k: Nat) { pred(k) implies pred(k.suc) }
    alt_induction(pred)
    forall(k: Nat) { pred(k) }
    pred(h)
    product[Nat](map(h.range, Nat.suc)) = h.factorial
}

/// Extending a signed doubling prefix appends the next representative.
theorem doubling_signed_product_prefix_suc(h: Nat, k: Nat) {
    doubling_signed_product_prefix(h, k.suc) =
        doubling_signed_product_prefix(h, k) *
            doubling_signed_index(h)(k).suc
} by {
    k.suc.range = k.range.append(k)
    k.range.append(k) = k.range + List.singleton(k)
    map_add[Nat, Nat](
        k.range, List.singleton(k), doubling_signed_index(h))
    map_singleton[Nat, Nat](doubling_signed_index(h), k)
    map(k.suc.range, doubling_signed_index(h)) =
        map(k.range, doubling_signed_index(h)).append(
            doubling_signed_index(h)(k))
    map_add[Nat, Nat](
        map(k.range, doubling_signed_index(h)),
        List.singleton(doubling_signed_index(h)(k)), Nat.suc)
    map_singleton[Nat, Nat](Nat.suc, doubling_signed_index(h)(k))
    map(map(k.suc.range, doubling_signed_index(h)), Nat.suc) =
        map(map(k.range, doubling_signed_index(h)), Nat.suc).append(
            doubling_signed_index(h)(k).suc)
    product_append[Nat](
        map(map(k.range, doubling_signed_index(h)), Nat.suc),
        List.singleton(doubling_signed_index(h)(k).suc))
    List.singleton(doubling_signed_index(h)(k).suc) = List.cons(
        doubling_signed_index(h)(k).suc, List.nil[Nat])
    product[Nat](List.nil[Nat]) = Nat.1
    doubling_signed_index(h)(k).suc * Nat.1 =
        doubling_signed_index(h)(k).suc
    product[Nat](List.singleton(doubling_signed_index(h)(k).suc)) =
        doubling_signed_index(h)(k).suc
    doubling_signed_product_prefix(h, k.suc) =
        doubling_signed_product_prefix(h, k) *
            doubling_signed_index(h)(k).suc
}

/// The signed doubling representatives have product `h!`.
theorem doubling_signed_product_eq_factorial(h: Nat) {
    doubling_signed_product(h) = h.factorial
} by {
    doubling_signed_indices_permute_range(h)
    is_permutation(
        map(h.range, doubling_signed_index(h)), h.range)
    permutation_preserves_mapped_product[Nat, Nat](
        map(h.range, doubling_signed_index(h)), h.range, Nat.suc)
    product[Nat](map(
        map(h.range, doubling_signed_index(h)), Nat.suc)) =
        product[Nat](map(h.range, Nat.suc))
    product_succ_range_eq_factorial(h)
    product[Nat](map(h.range, Nat.suc)) = h.factorial
    doubling_signed_product(h) = h.factorial
}

/// An even power of the predecessor of a prime is congruent to one modulo the
/// prime.
theorem prime_pred_even_power_congr_one(p: Nat, q: Nat) {
    p.is_prime implies (p - Nat.1).pow(Nat.2 * q).congr_mod(Nat.1, p)
} by {
    if p.is_prime {
        prime_pred_self_inverse_congr(p)
        ((p - Nat.1) * (p - Nat.1)).congr_mod(Nat.1, p)
        exp_add(p - Nat.1, Nat.1, Nat.1)
        (p - Nat.1).pow(Nat.1 + Nat.1) =
            (p - Nat.1).pow(Nat.1) * (p - Nat.1).pow(Nat.1)
        exp_one(p - Nat.1)
        (p - Nat.1).pow(Nat.1) = p - Nat.1
        Nat.1 + Nat.1 = Nat.2
        (p - Nat.1).pow(Nat.2) = (p - Nat.1) * (p - Nat.1)
        (p - Nat.1).pow(Nat.2).congr_mod(Nat.1, p)
        congr_mod_pow((p - Nat.1).pow(Nat.2), Nat.1, p, q)
        (p - Nat.1).pow(Nat.2).pow(q).congr_mod(Nat.1.pow(q), p)
        exp_mul(p - Nat.1, Nat.2, q)
        (p - Nat.1).pow(Nat.2 * q) = (p - Nat.1).pow(Nat.2).pow(q)
        one_exp(q)
        Nat.1.pow(q) = Nat.1
        (p - Nat.1).pow(Nat.2 * q).congr_mod(Nat.1, p)
    }
}

/// An odd power of the predecessor of a prime is congruent to the predecessor
/// modulo the prime.
theorem prime_pred_odd_power_congr_pred(p: Nat, q: Nat) {
    p.is_prime implies
        (p - Nat.1).pow(Nat.2 * q + Nat.1).congr_mod(p - Nat.1, p)
} by {
    if p.is_prime {
        prime_pred_even_power_congr_one(p, q)
        (p - Nat.1).pow(Nat.2 * q).congr_mod(Nat.1, p)
        congr_mod_refl(p - Nat.1, p)
        congr_mod_mul(
            (p - Nat.1).pow(Nat.2 * q), p - Nat.1,
            Nat.1, p - Nat.1, p)
        ((p - Nat.1).pow(Nat.2 * q) * (p - Nat.1)).congr_mod(
            Nat.1 * (p - Nat.1), p)
        exp_add(p - Nat.1, Nat.2 * q, Nat.1)
        (p - Nat.1).pow(Nat.2 * q + Nat.1) =
            (p - Nat.1).pow(Nat.2 * q) * (p - Nat.1).pow(Nat.1)
        exp_one(p - Nat.1)
        (p - Nat.1).pow(Nat.1) = p - Nat.1
        Nat.1 * (p - Nat.1) = p - Nat.1
        (p - Nat.1).pow(Nat.2 * q + Nat.1).congr_mod(p - Nat.1, p)
    }
}

/// An odd number whose half-predecessor is even has remainder one modulo four.
theorem double_add_one_mod_four_of_even(h: Nat) {
    Nat.2.divides(h) implies (Nat.2 * h + Nat.1).mod(Nat.4) = Nat.1
} by {
    if Nat.2.divides(h) {
        Nat.2.divides(h) = exists(q: Nat) { Nat.2 * q = h }
        let q: Nat satisfy { Nat.2 * q = h }
        Nat.2 * h = Nat.2 * (Nat.2 * q)
        Nat.2 * (Nat.2 * q) = (Nat.2 * Nat.2) * q
        Nat.2 * Nat.2 = Nat.4
        Nat.4 * q = q * Nat.4
        Nat.2 * h = q * Nat.4
        Nat.2 * h + Nat.1 = q * Nat.4 + Nat.1
        mod_add_mul(q, Nat.4, Nat.1)
        (q * Nat.4 + Nat.1).mod(Nat.4) = Nat.1.mod(Nat.4)
        (Nat.2 * h + Nat.1).mod(Nat.4) = Nat.1.mod(Nat.4)
        lt_add_suc(Nat.1, Nat.2)
        Nat.1 + Nat.3 = Nat.4
        Nat.1 < Nat.4
        small_mod(Nat.1, Nat.4)
        Nat.1.mod(Nat.4) = Nat.1
        (Nat.2 * h + Nat.1).mod(Nat.4) = Nat.1
    }
}

/// An odd number whose half-predecessor is odd has remainder three modulo four.
theorem double_add_one_mod_four_of_odd(h: Nat) {
    not Nat.2.divides(h) implies (Nat.2 * h + Nat.1).mod(Nat.4) = Nat.3
} by {
    if not Nat.2.divides(h) {
        Nat.2.divides(h) = exists(k: Nat) { Nat.2 * k = h }
        add_mod(h, Nat.2)
        let q: Nat satisfy { q * Nat.2 + h.mod(Nat.2) = h }
        alt_suc_ne_zero(Nat.1)
        Nat.2 != Nat.0
        mod_lt(h, Nat.2)
        if h.mod(Nat.2) = Nat.0 {
            Nat.2 * q = h
            Nat.2.divides(h)
            false
        }
        lt_suc_right(h.mod(Nat.2), Nat.1)
        if h.mod(Nat.2) != Nat.1 {
            h.mod(Nat.2) < Nat.1
            lt_suc_right(h.mod(Nat.2), Nat.0)
            not_lt_zero(h.mod(Nat.2))
            h.mod(Nat.2) = Nat.0
            false
        }
        h.mod(Nat.2) = Nat.1
        h = q * Nat.2 + Nat.1
        Nat.2 * h = Nat.2 * (q * Nat.2 + Nat.1)
        Nat.2 * (q * Nat.2 + Nat.1) = Nat.2 * (q * Nat.2) + Nat.2
        Nat.2 * (q * Nat.2) = (Nat.2 * q) * Nat.2
        Nat.2 * q = q * Nat.2
        (q * Nat.2) * Nat.2 = q * (Nat.2 * Nat.2)
        Nat.2 * Nat.2 = Nat.4
        Nat.2 * (q * Nat.2) = q * Nat.4
        Nat.2 + Nat.1 = Nat.3
        Nat.2 * h + Nat.1 = q * Nat.4 + Nat.3
        mod_add_mul(q, Nat.4, Nat.3)
        (q * Nat.4 + Nat.3).mod(Nat.4) = Nat.3.mod(Nat.4)
        (Nat.2 * h + Nat.1).mod(Nat.4) = Nat.3.mod(Nat.4)
        lt_add_suc(Nat.3, Nat.0)
        Nat.3 + Nat.1 = Nat.4
        Nat.3 < Nat.4
        small_mod(Nat.3, Nat.4)
        Nat.3.mod(Nat.4) = Nat.3
        (Nat.2 * h + Nat.1).mod(Nat.4) = Nat.3
    }
}

/// The half-predecessor of an odd number is even exactly when the number is
/// congruent to one modulo four.
theorem half_pred_even_iff_congr_one_mod_four(p: Nat, h: Nat) {
    p = Nat.2 * h + Nat.1 implies
        (Nat.2.divides(h) = p.congr_mod(Nat.1, Nat.4))
} by {
    if p = Nat.2 * h + Nat.1 {
        lt_add_suc(Nat.1, Nat.2)
        Nat.1 + Nat.3 = Nat.4
        Nat.1 < Nat.4
        small_mod(Nat.1, Nat.4)
        Nat.1.mod(Nat.4) = Nat.1
        p.congr_mod(Nat.1, Nat.4) =
            (p.mod(Nat.4) = Nat.1.mod(Nat.4))
        if Nat.2.divides(h) {
            double_add_one_mod_four_of_even(h)
            p.mod(Nat.4) = (Nat.2 * h + Nat.1).mod(Nat.4)
            p.mod(Nat.4) = Nat.1
            p.mod(Nat.4) = Nat.1.mod(Nat.4)
            p.congr_mod(Nat.1, Nat.4)
            Nat.2.divides(h) = p.congr_mod(Nat.1, Nat.4)
        } else {
            double_add_one_mod_four_of_odd(h)
            p.mod(Nat.4) = (Nat.2 * h + Nat.1).mod(Nat.4)
            p.mod(Nat.4) = Nat.3
            p.mod(Nat.4) != Nat.1.mod(Nat.4)
            not p.congr_mod(Nat.1, Nat.4)
            Nat.2.divides(h) = p.congr_mod(Nat.1, Nat.4)
        }
    }
}

/// The half-predecessor of an odd number is odd exactly when the number is
/// congruent to three modulo four.
theorem half_pred_odd_iff_congr_three_mod_four(p: Nat, h: Nat) {
    p = Nat.2 * h + Nat.1 implies
        ((not Nat.2.divides(h)) = p.congr_mod(Nat.3, Nat.4))
} by {
    if p = Nat.2 * h + Nat.1 {
        lt_add_suc(Nat.3, Nat.0)
        Nat.3 + Nat.1 = Nat.4
        Nat.3 < Nat.4
        small_mod(Nat.3, Nat.4)
        Nat.3.mod(Nat.4) = Nat.3
        p.congr_mod(Nat.3, Nat.4) =
            (p.mod(Nat.4) = Nat.3.mod(Nat.4))
        if Nat.2.divides(h) {
            double_add_one_mod_four_of_even(h)
            p.mod(Nat.4) = (Nat.2 * h + Nat.1).mod(Nat.4)
            p.mod(Nat.4) = Nat.1
            p.mod(Nat.4) != Nat.3.mod(Nat.4)
            not p.congr_mod(Nat.3, Nat.4)
            (not Nat.2.divides(h)) = p.congr_mod(Nat.3, Nat.4)
        } else {
            double_add_one_mod_four_of_odd(h)
            p.mod(Nat.4) = (Nat.2 * h + Nat.1).mod(Nat.4)
            p.mod(Nat.4) = Nat.3
            p.mod(Nat.4) = Nat.3.mod(Nat.4)
            p.congr_mod(Nat.3, Nat.4)
            (not Nat.2.divides(h)) = p.congr_mod(Nat.3, Nat.4)
        }
    }
}

/// The factorial of twice a positive half-size splits into its lower-half
/// factorial and its descending upper-half product.
theorem double_factorial_split_halves(h: Nat) {
    (Nat.2 * h.suc).factorial =
        h.suc.factorial * falling_product(Nat.2 * h.suc, h)
} by {
    lt_suc(h)
    alt_suc_ne_zero(Nat.1)
    Nat.2 != Nat.0
    lte_mul(h.suc, Nat.2)
    h.suc <= h.suc * Nat.2
    h.suc * Nat.2 = Nat.2 * h.suc
    h.suc <= Nat.2 * h.suc
    lt_and_lte(h, h.suc, Nat.2 * h.suc)
    h < Nat.2 * h.suc
    falling_product_mul_factorial_complement(Nat.2 * h.suc, h)
    Nat.2 * h.suc = h.suc + h.suc
    add_imp_sub_left(h.suc, h.suc, Nat.2 * h.suc)
    Nat.2 * h.suc - h.suc = h.suc
    falling_product(Nat.2 * h.suc, h) * h.suc.factorial =
        (Nat.2 * h.suc).factorial
    h.suc.factorial * falling_product(Nat.2 * h.suc, h) =
        falling_product(Nat.2 * h.suc, h) * h.suc.factorial
    (Nat.2 * h.suc).factorial =
        h.suc.factorial * falling_product(Nat.2 * h.suc, h)
}

/// The factorial below an odd number splits into the lower positive half and
/// the descending upper positive half.
theorem odd_pred_factorial_split_halves(p: Nat, h: Nat) {
    p = Nat.2 * h + Nat.1 and h != Nat.0 implies
        (p - Nat.1).factorial =
            h.factorial * falling_product(p - Nat.1, h - Nat.1)
} by {
    if p = Nat.2 * h + Nat.1 and h != Nat.0 {
        zero_or_suc(h)
        let q: Nat satisfy { h = q.suc }
        double_factorial_split_halves(q)
        suc_sub_one(q)
        h - Nat.1 = q
        p - Nat.1 = Nat.2 * h
        (p - Nat.1).factorial =
            h.factorial * falling_product(p - Nat.1, h - Nat.1)
    }
}

/// Subtraction from a positive modulus is congruent to multiplication by
/// minus one, represented by the predecessor of the modulus.
theorem modulus_sub_congr_pred_mul(p: Nat, i: Nat) {
    p != Nat.0 and i <= p implies (p - i).congr_mod((p - Nat.1) * i, p)
} by {
    if p != Nat.0 and i <= p {
        add_sub(p, i)
        p - i + i = p
        zero_or_suc(p)
        let q: Nat satisfy { p = q.suc }
        suc_sub_one(q)
        p - Nat.1 = q
        (p - Nat.1) * i + i = (p - Nat.1) * i + Nat.1 * i
        (p - Nat.1) * i + Nat.1 * i = (p - Nat.1 + Nat.1) * i
        p - Nat.1 + Nat.1 = p
        (p - Nat.1) * i + i = p * i
        divides_self(p)
        p.divides(p)
        divides_mul(p, i, p)
        p.divides(p * i)
        divides_imp_congr_zero(p, p)
        p.congr_mod(Nat.0, p)
        divides_imp_congr_zero(p, p * i)
        (p * i).congr_mod(Nat.0, p)
        congr_mod_symm(p * i, Nat.0, p)
        Nat.0.congr_mod(p * i, p)
        congr_mod_trans(p, Nat.0, p * i, p)
        p.congr_mod(p * i, p)
        (p - i + i).congr_mod((p - Nat.1) * i + i, p)
        congr_mod_add_cancel_right_pos(p - i, (p - Nat.1) * i, i, p)
        (p - i).congr_mod((p - Nat.1) * i, p)
    }
}

/// Each zero-based signed doubling representative is congruent to its
/// unsigned double, with one predecessor factor exactly in the upper case.
theorem doubling_signed_index_suc_factor_congr(
    p: Nat, h: Nat, i: Nat
) {
    p = Nat.2 * h + Nat.1 and i < h implies
        doubling_signed_index(h)(i).suc.congr_mod(
            if Nat.2 * i.suc <= h {
                Nat.2 * i.suc
            } else {
                (p - Nat.1) * (Nat.2 * i.suc)
            }, p)
} by {
    if p = Nat.2 * h + Nat.1 and i < h {
        lt_imp_lte_suc(i, h)
        i.suc <= h
        alt_suc_ne_zero(i)
        i.suc != Nat.0
        Nat.0 < i.suc
        doubling_signed_representative_positive_bounded(h, i.suc)
        Nat.0 < doubling_signed_representative(h, i.suc)
        doubling_signed_index(h)(i) =
            doubling_signed_representative(h, i.suc) - Nat.1
        add_sub(doubling_signed_representative(h, i.suc), Nat.1)
        (doubling_signed_representative(h, i.suc) - Nat.1) + Nat.1 =
            doubling_signed_representative(h, i.suc)
        add_one_right(doubling_signed_index(h)(i))
        doubling_signed_index(h)(i).suc =
            doubling_signed_index(h)(i) + Nat.1
        doubling_signed_index(h)(i).suc =
            doubling_signed_representative(h, i.suc)
        if Nat.2 * i.suc <= h {
            doubling_signed_representative(h, i.suc) = Nat.2 * i.suc
            congr_mod_refl(Nat.2 * i.suc, p)
            doubling_signed_index(h)(i).suc.congr_mod(
                Nat.2 * i.suc, p)
            doubling_signed_index(h)(i).suc.congr_mod(
                if Nat.2 * i.suc <= h {
                    Nat.2 * i.suc
                } else {
                    (p - Nat.1) * (Nat.2 * i.suc)
                }, p)
        } else {
            doubling_signed_representative(h, i.suc) =
                Nat.2 * h + Nat.1 - Nat.2 * i.suc
            doubling_signed_index(h)(i).suc = p - Nat.2 * i.suc
            add_one_right(Nat.2 * h)
            p = (Nat.2 * h).suc
            alt_suc_ne_zero(Nat.2 * h)
            p != Nat.0
            lte_mul_both(Nat.2, i.suc, h)
            Nat.2 * i.suc <= Nat.2 * h
            sum_lte(Nat.2 * h, Nat.0, Nat.2 * h, Nat.1)
            Nat.2 * h <= Nat.2 * h + Nat.1
            lte_trans(Nat.2 * i.suc, Nat.2 * h, Nat.2 * h + Nat.1)
            Nat.2 * i.suc <= Nat.2 * h + Nat.1
            Nat.2 * i.suc <= p
            modulus_sub_congr_pred_mul(p, Nat.2 * i.suc)
            (p - Nat.2 * i.suc).congr_mod(
                (p - Nat.1) * (Nat.2 * i.suc), p)
            doubling_signed_index(h)(i).suc.congr_mod(
                (p - Nat.1) * (Nat.2 * i.suc), p)
            (if Nat.2 * i.suc <= h {
                Nat.2 * i.suc
            } else {
                (p - Nat.1) * (Nat.2 * i.suc)
            }) = (p - Nat.1) * (Nat.2 * i.suc)
            doubling_signed_index(h)(i).suc.congr_mod(
                if Nat.2 * i.suc <= h {
                    Nat.2 * i.suc
                } else {
                    (p - Nat.1) * (Nat.2 * i.suc)
                }, p)
        }
    }
}

/// A descending prefix below a positive modulus contributes one predecessor
/// factor, hence one minus sign, for each factorial factor.
theorem falling_product_pred_sign(p: Nat, k: Nat) {
    p != Nat.0 and k.suc < p implies
        falling_product(p - Nat.1, k).congr_mod(
            (p - Nat.1).pow(k.suc) * k.suc.factorial, p)
} by {
    define pred(j: Nat) -> Bool {
        j.suc < p implies falling_product(p - Nat.1, j).congr_mod(
            (p - Nat.1).pow(j.suc) * j.suc.factorial, p)
    }

    if Nat.1 < p {
        falling_product(p - Nat.1, Nat.0) = p - Nat.1
        exp_one(p - Nat.1)
        factorial_one
        (p - Nat.1).pow(Nat.1) * Nat.1.factorial = p - Nat.1
        congr_mod_refl(p - Nat.1, p)
        falling_product(p - Nat.1, Nat.0).congr_mod(
            (p - Nat.1).pow(Nat.0.suc) * Nat.0.suc.factorial, p)
    }
    pred(Nat.0)

    forall(j: Nat) {
        if pred(j) {
            if j.suc.suc < p {
                j.suc < p
                falling_product(p - Nat.1, j).congr_mod(
                    (p - Nat.1).pow(j.suc) * j.suc.factorial, p)
                falling_product_suc(p - Nat.1, j)
                let d: Nat satisfy { j.suc.suc + d = p }
                p - Nat.1 = j.suc + d
                (p - Nat.1) - j.suc = d
                p - j.suc.suc = d
                (p - Nat.1) - j.suc = p - j.suc.suc
                modulus_sub_congr_pred_mul(p, j.suc.suc)
                (p - j.suc.suc).congr_mod(
                    (p - Nat.1) * j.suc.suc, p)
                ((p - Nat.1) - j.suc).congr_mod(
                    (p - Nat.1) * j.suc.suc, p)
                congr_mod_mul(
                    falling_product(p - Nat.1, j),
                    (p - Nat.1) - j.suc,
                    (p - Nat.1).pow(j.suc) * j.suc.factorial,
                    (p - Nat.1) * j.suc.suc, p)
                falling_product(p - Nat.1, j.suc).congr_mod(
                    ((p - Nat.1).pow(j.suc) * j.suc.factorial) *
                        ((p - Nat.1) * j.suc.suc), p)
                (p - Nat.1).pow(j.suc.suc) =
                    (p - Nat.1).pow(j.suc) * (p - Nat.1)
                factorial_step(j.suc)
                j.suc.suc.factorial = j.suc.suc * j.suc.factorial
                nat_mul_swap_middle(
                    (p - Nat.1).pow(j.suc), j.suc.factorial,
                    p - Nat.1, j.suc.suc)
                ((p - Nat.1).pow(j.suc) * j.suc.factorial) *
                        ((p - Nat.1) * j.suc.suc) =
                    ((p - Nat.1).pow(j.suc) * (p - Nat.1)) *
                        (j.suc.factorial * j.suc.suc)
                j.suc.factorial * j.suc.suc =
                    j.suc.suc * j.suc.factorial
                j.suc.factorial * j.suc.suc = j.suc.suc.factorial
                ((p - Nat.1).pow(j.suc) * j.suc.factorial) *
                        ((p - Nat.1) * j.suc.suc) =
                    (p - Nat.1).pow(j.suc.suc) * j.suc.suc.factorial
                falling_product(p - Nat.1, j.suc).congr_mod(
                    (p - Nat.1).pow(j.suc.suc) * j.suc.suc.factorial, p)
            }
            pred(j.suc)
        }
    }

    forall(j: Nat) { pred(j) implies pred(j.suc) }
    pred(Nat.0) and forall(j: Nat) { pred(j) implies pred(j.suc) }
    alt_induction(pred)
    forall(j: Nat) { pred(j) }
    pred(k)
}

/// The descending upper half below an odd modulus contributes the sign
/// represented by one predecessor power, together with the lower factorial.
theorem odd_upper_half_product_sign(p: Nat, h: Nat) {
    p = Nat.2 * h + Nat.1 and h != Nat.0 implies
        falling_product(p - Nat.1, h - Nat.1).congr_mod(
            (p - Nat.1).pow(h) * h.factorial, p)
} by {
    if p = Nat.2 * h + Nat.1 and h != Nat.0 {
        zero_or_suc(h)
        let q: Nat satisfy { h = q.suc }
        suc_sub_one(q)
        h - Nat.1 = q
        add_one_right(Nat.2 * h)
        p = (Nat.2 * h).suc
        alt_suc_ne_zero(Nat.2 * h)
        p != Nat.0
        alt_suc_ne_zero(Nat.1)
        Nat.2 != Nat.0
        lte_mul(h, Nat.2)
        h <= h * Nat.2
        h * Nat.2 = Nat.2 * h
        h <= Nat.2 * h
        lt_suc(Nat.2 * h)
        Nat.2 * h < p
        lte_and_lt(h, Nat.2 * h, p)
        h < p
        q.suc < p
        p != Nat.0 and q.suc < p
        falling_product_pred_sign(p, q)
        falling_product(p - Nat.1, q).congr_mod(
            (p - Nat.1).pow(q.suc) * q.suc.factorial, p)
        falling_product(p - Nat.1, h - Nat.1) =
            falling_product(p - Nat.1, q)
        (p - Nat.1).pow(q.suc) * q.suc.factorial =
            (p - Nat.1).pow(h) * h.factorial
        falling_product(p - Nat.1, h - Nat.1).congr_mod(
            (p - Nat.1).pow(h) * h.factorial, p)
    }
}

/// Wilson's theorem expresses the signed square of the lower-half factorial
/// as the predecessor of an odd prime modulus.
theorem prime_half_factorial_signed_square_congr_pred(p: Nat, h: Nat) {
    p.is_prime and p = Nat.2 * h + Nat.1 and h != Nat.0 implies
        ((p - Nat.1).pow(h) * (h.factorial * h.factorial)).congr_mod(
            p - Nat.1, p)
} by {
    if p.is_prime and p = Nat.2 * h + Nat.1 and h != Nat.0 {
        odd_pred_factorial_split_halves(p, h)
        (p - Nat.1).factorial =
            h.factorial * falling_product(p - Nat.1, h - Nat.1)
        odd_upper_half_product_sign(p, h)
        falling_product(p - Nat.1, h - Nat.1).congr_mod(
            (p - Nat.1).pow(h) * h.factorial, p)
        congr_mod_refl(h.factorial, p)
        congr_mod_mul(
            h.factorial, falling_product(p - Nat.1, h - Nat.1),
            h.factorial, (p - Nat.1).pow(h) * h.factorial, p)
        (h.factorial * falling_product(p - Nat.1, h - Nat.1)).congr_mod(
            h.factorial * ((p - Nat.1).pow(h) * h.factorial), p)
        h.factorial * ((p - Nat.1).pow(h) * h.factorial) =
            (p - Nat.1).pow(h) * (h.factorial * h.factorial)
        (p - Nat.1).factorial.congr_mod(
            (p - Nat.1).pow(h) * (h.factorial * h.factorial), p)
        congr_mod_symm(
            (p - Nat.1).factorial,
            (p - Nat.1).pow(h) * (h.factorial * h.factorial), p)
        ((p - Nat.1).pow(h) * (h.factorial * h.factorial)).congr_mod(
            (p - Nat.1).factorial, p)
        prime_imp_wilson_factorial_congr(p)
        (p - Nat.1).factorial.congr_mod(p - Nat.1, p)
        congr_mod_trans(
            (p - Nat.1).pow(h) * (h.factorial * h.factorial),
            (p - Nat.1).factorial, p - Nat.1, p)
        ((p - Nat.1).pow(h) * (h.factorial * h.factorial)).congr_mod(
            p - Nat.1, p)
    }
}

/// For an even half-exponent, the lower-half factorial is an explicit square
/// root of minus one modulo the odd prime.
theorem prime_even_half_factorial_square_congr_pred(p: Nat, h: Nat) {
    p.is_prime and p = Nat.2 * h + Nat.1 and h != Nat.0 and
        Nat.2.divides(h) implies
        h.factorial.pow(Nat.2).congr_mod(p - Nat.1, p)
} by {
    if p.is_prime and p = Nat.2 * h + Nat.1 and h != Nat.0 and
            Nat.2.divides(h) {
        prime_half_factorial_signed_square_congr_pred(p, h)
        ((p - Nat.1).pow(h) * (h.factorial * h.factorial)).congr_mod(
            p - Nat.1, p)
        Nat.2.divides(h) = exists(q: Nat) { Nat.2 * q = h }
        let q: Nat satisfy { Nat.2 * q = h }
        prime_pred_even_power_congr_one(p, q)
        (p - Nat.1).pow(Nat.2 * q).congr_mod(Nat.1, p)
        (p - Nat.1).pow(h).congr_mod(Nat.1, p)
        congr_mod_refl(h.factorial * h.factorial, p)
        congr_mod_mul(
            (p - Nat.1).pow(h), h.factorial * h.factorial,
            Nat.1, h.factorial * h.factorial, p)
        ((p - Nat.1).pow(h) * (h.factorial * h.factorial)).congr_mod(
            Nat.1 * (h.factorial * h.factorial), p)
        Nat.1 * (h.factorial * h.factorial) = h.factorial * h.factorial
        ((p - Nat.1).pow(h) * (h.factorial * h.factorial)).congr_mod(
            h.factorial * h.factorial, p)
        congr_mod_symm(
            (p - Nat.1).pow(h) * (h.factorial * h.factorial),
            h.factorial * h.factorial, p)
        (h.factorial * h.factorial).congr_mod(
            (p - Nat.1).pow(h) * (h.factorial * h.factorial), p)
        congr_mod_trans(
            h.factorial * h.factorial,
            (p - Nat.1).pow(h) * (h.factorial * h.factorial),
            p - Nat.1, p)
        (h.factorial * h.factorial).congr_mod(p - Nat.1, p)
        sq_eq_mul(h.factorial)
        h.factorial.pow(Nat.2) = h.factorial * h.factorial
        h.factorial.pow(Nat.2).congr_mod(p - Nat.1, p)
    }
}

/// Minus one is a quadratic residue modulo an odd prime whose half-exponent
/// is even.
theorem prime_even_half_pred_is_quadratic_residue(p: Nat, h: Nat) {
    p.is_prime and p = Nat.2 * h + Nat.1 and h != Nat.0 and
        Nat.2.divides(h) implies is_quadratic_residue_mod(p - Nat.1, p)
} by {
    if p.is_prime and p = Nat.2 * h + Nat.1 and h != Nat.0 and
            Nat.2.divides(h) {
        prime_even_half_factorial_square_congr_pred(p, h)
        h.factorial.pow(Nat.2).congr_mod(p - Nat.1, p)
        quadratic_residue_of_square_congr(h.factorial, p - Nat.1, p)
        is_quadratic_residue_mod(p - Nat.1, p)
    }
}

/// Every natural not divisible by two has an odd decomposition.
theorem odd_decomp_of_not_two_divides(h: Nat) {
    not Nat.2.divides(h) implies exists(q: Nat) {
        h = Nat.2 * q + Nat.1
    }
} by {
    if not Nat.2.divides(h) {
        Nat.2.divides(h) = exists(k: Nat) { Nat.2 * k = h }
        add_mod(h, Nat.2)
        let q: Nat satisfy { q * Nat.2 + h.mod(Nat.2) = h }
        alt_suc_ne_zero(Nat.1)
        Nat.2 != Nat.0
        mod_lt(h, Nat.2)
        if h.mod(Nat.2) = Nat.0 {
            Nat.2 * q = h
            Nat.2.divides(h)
            false
        }
        lt_suc_right(h.mod(Nat.2), Nat.1)
        if h.mod(Nat.2) != Nat.1 {
            h.mod(Nat.2) < Nat.1
            lt_suc_right(h.mod(Nat.2), Nat.0)
            not_lt_zero(h.mod(Nat.2))
            h.mod(Nat.2) = Nat.0
            false
        }
        h.mod(Nat.2) = Nat.1
        h = q * Nat.2 + Nat.1
        q * Nat.2 = Nat.2 * q
        h = Nat.2 * q + Nat.1
        exists(r: Nat) { h = Nat.2 * r + Nat.1 }
    }
}

/// Minus one is not a quadratic residue modulo an odd prime whose
/// half-exponent is odd.
theorem prime_odd_half_pred_not_quadratic_residue(p: Nat, h: Nat) {
    p.is_prime and p = Nat.2 * h + Nat.1 and not Nat.2.divides(h) implies
        not is_quadratic_residue_mod(p - Nat.1, p)
} by {
    if p.is_prime and p = Nat.2 * h + Nat.1 and not Nat.2.divides(h) {
        odd_decomp_of_not_two_divides(h)
        let q: Nat satisfy { h = Nat.2 * q + Nat.1 }
        if h = Nat.0 {
            divides_zero(Nat.2)
            Nat.2.divides(h)
            false
        }
        h != Nat.0
        alt_suc_ne_zero(Nat.1)
        Nat.2 != Nat.0
        lte_mul(Nat.2, h)
        Nat.2 <= Nat.2 * h
        p - Nat.1 = Nat.2 * h
        Nat.2 <= p - Nat.1
        lt_suc(Nat.1)
        Nat.1 < Nat.2
        lte_imp_not_lt(Nat.2, p - Nat.1)
        p - Nat.1 != Nat.1
        Nat.1 < p
        sub_one_lt(p)
        p - Nat.1 < p
        small_mod(p - Nat.1, p)
        (p - Nat.1).mod(p) = p - Nat.1
        sub_pos(p, Nat.1)
        Nat.0 < p - Nat.1
        p - Nat.1 != Nat.0
        mod_of_zero(p)
        Nat.0.mod(p) = Nat.0
        not (p - Nat.1).congr_mod(Nat.0, p)
        if is_quadratic_residue_mod(p - Nat.1, p) {
            euler_criterion_quadratic_residue_forward(p, h, p - Nat.1)
            (p - Nat.1).pow(h).congr_mod(Nat.1, p)
            prime_pred_odd_power_congr_pred(p, q)
            (p - Nat.1).pow(Nat.2 * q + Nat.1).congr_mod(p - Nat.1, p)
            (p - Nat.1).pow(h).congr_mod(p - Nat.1, p)
            congr_mod_symm((p - Nat.1).pow(h), Nat.1, p)
            Nat.1.congr_mod((p - Nat.1).pow(h), p)
            congr_mod_trans(Nat.1, (p - Nat.1).pow(h), p - Nat.1, p)
            Nat.1.congr_mod(p - Nat.1, p)
            small_mod(Nat.1, p)
            Nat.1.mod(p) = Nat.1
            Nat.1.congr_mod(p - Nat.1, p) =
                (Nat.1.mod(p) = (p - Nat.1).mod(p))
            Nat.1.mod(p) = (p - Nat.1).mod(p)
            Nat.1 = p - Nat.1
            false
        }
        not is_quadratic_residue_mod(p - Nat.1, p)
    }
}

/// Minus one is a quadratic residue modulo an odd prime exactly when the
/// prime's half-exponent is even.
theorem prime_pred_quadratic_residue_iff_half_even(p: Nat, h: Nat) {
    p.is_prime and p = Nat.2 * h + Nat.1 implies
        (is_quadratic_residue_mod(p - Nat.1, p) = Nat.2.divides(h))
} by {
    if p.is_prime and p = Nat.2 * h + Nat.1 {
        if h = Nat.0 {
            p = Nat.1
            Nat.1 < p
            lt_not_ref(Nat.1)
            false
        }
        h != Nat.0
        if Nat.2.divides(h) {
            prime_even_half_pred_is_quadratic_residue(p, h)
            is_quadratic_residue_mod(p - Nat.1, p)
            is_quadratic_residue_mod(p - Nat.1, p) = Nat.2.divides(h)
        } else {
            prime_odd_half_pred_not_quadratic_residue(p, h)
            not is_quadratic_residue_mod(p - Nat.1, p)
            is_quadratic_residue_mod(p - Nat.1, p) = Nat.2.divides(h)
        }
    }
}

/// Minus one is a quadratic residue modulo an odd prime exactly when the
/// prime is congruent to one modulo four.
theorem prime_pred_quadratic_residue_iff_congr_one_mod_four(p: Nat, h: Nat) {
    p.is_prime and p = Nat.2 * h + Nat.1 implies
        (is_quadratic_residue_mod(p - Nat.1, p) =
            p.congr_mod(Nat.1, Nat.4))
} by {
    if p.is_prime and p = Nat.2 * h + Nat.1 {
        prime_pred_quadratic_residue_iff_half_even(p, h)
        is_quadratic_residue_mod(p - Nat.1, p) = Nat.2.divides(h)
        half_pred_even_iff_congr_one_mod_four(p, h)
        Nat.2.divides(h) = p.congr_mod(Nat.1, Nat.4)
        is_quadratic_residue_mod(p - Nat.1, p) =
            p.congr_mod(Nat.1, Nat.4)
    }
}

/// The predecessor of an odd prime is not congruent to zero modulo that
/// prime.
theorem odd_prime_pred_not_congr_zero(p: Nat, h: Nat) {
    p.is_prime and p = Nat.2 * h + Nat.1 implies
        not (p - Nat.1).congr_mod(Nat.0, p)
} by {
    if p.is_prime and p = Nat.2 * h + Nat.1 {
        if h = Nat.0 {
            p = Nat.1
            Nat.1 < p
            lt_not_ref(Nat.1)
            false
        }
        h != Nat.0
        alt_suc_ne_zero(Nat.1)
        Nat.2 != Nat.0
        lte_mul(Nat.2, h)
        Nat.2 <= Nat.2 * h
        p - Nat.1 = Nat.2 * h
        Nat.2 <= p - Nat.1
        Nat.1 < p
        sub_one_lt(p)
        p - Nat.1 < p
        small_mod(p - Nat.1, p)
        (p - Nat.1).mod(p) = p - Nat.1
        sub_pos(p, Nat.1)
        Nat.0 < p - Nat.1
        p - Nat.1 != Nat.0
        mod_of_zero(p)
        Nat.0.mod(p) = Nat.0
        not (p - Nat.1).congr_mod(Nat.0, p)
    }
}

/// The Legendre symbol of minus one modulo an odd prime is determined by the
/// parity of the prime's half-exponent.
theorem prime_pred_legendre_symbol_by_half_parity(p: Nat, h: Nat) {
    p.is_prime and p = Nat.2 * h + Nat.1 implies
        legendre_symbol(p - Nat.1, p) =
            if Nat.2.divides(h) { Int.1 } else { -Int.1 }
} by {
    if p.is_prime and p = Nat.2 * h + Nat.1 {
        odd_prime_pred_not_congr_zero(p, h)
        not (p - Nat.1).congr_mod(Nat.0, p)
        if Nat.2.divides(h) {
            prime_pred_quadratic_residue_iff_half_even(p, h)
            is_quadratic_residue_mod(p - Nat.1, p)
            legendre_symbol_one_of_nonzero_residue(p - Nat.1, p)
            legendre_symbol(p - Nat.1, p) = Int.1
            legendre_symbol(p - Nat.1, p) =
                if Nat.2.divides(h) { Int.1 } else { -Int.1 }
        } else {
            prime_pred_quadratic_residue_iff_half_even(p, h)
            not is_quadratic_residue_mod(p - Nat.1, p)
            quadratic_nonresidue_of_not_residue(p - Nat.1, p)
            is_quadratic_nonresidue_mod(p - Nat.1, p)
            legendre_symbol_neg_one_of_nonzero_nonresidue(p - Nat.1, p)
            legendre_symbol(p - Nat.1, p) = -Int.1
            legendre_symbol(p - Nat.1, p) =
                if Nat.2.divides(h) { Int.1 } else { -Int.1 }
        }
    }
}

/// The Legendre symbol of minus one modulo an odd prime is one precisely in
/// the residue class one modulo four, and minus one otherwise.
theorem prime_pred_legendre_symbol_by_mod_four(p: Nat, h: Nat) {
    p.is_prime and p = Nat.2 * h + Nat.1 implies
        legendre_symbol(p - Nat.1, p) =
            if p.congr_mod(Nat.1, Nat.4) { Int.1 } else { -Int.1 }
} by {
    if p.is_prime and p = Nat.2 * h + Nat.1 {
        prime_pred_legendre_symbol_by_half_parity(p, h)
        half_pred_even_iff_congr_one_mod_four(p, h)
        Nat.2.divides(h) = p.congr_mod(Nat.1, Nat.4)
        legendre_symbol(p - Nat.1, p) =
            if Nat.2.divides(h) { Int.1 } else { -Int.1 }
        legendre_symbol(p - Nat.1, p) =
            if p.congr_mod(Nat.1, Nat.4) { Int.1 } else { -Int.1 }
    }
}

/// A quotient is fixed by any decomposition with a remainder below the
/// modulus.
lemma nat_div_of_decomp_eq(a: Nat, q: Nat, r: Nat, m: Nat) {
    r < m and a = q * m + r implies a.div(m) = q
} by {
    if r < m and a = q * m + r {
        let d = a.div(m)
        div_mod_decomp(a, m)
        d * m + a.mod(m) = a
        mod_lt(a, m)
        if d < q {
            lt_diff(d, q)
            let c: Nat satisfy { d + c = q and c != Nat.0 }
            q * m = (d + c) * m
            q * m = d * m + c * m
            d * m + a.mod(m) = d * m + (c * m + r)
            add_cancels_left(d * m, a.mod(m), c * m + r)
            a.mod(m) = c * m + r
            lte_mul(m, c)
            m <= c * m
            c * m <= c * m + r
            lte_trans(m, c * m, c * m + r)
            m <= c * m + r
            m <= a.mod(m)
            lte_imp_not_lt(m, a.mod(m))
            false
        }
        not d < q
        if q < d {
            lt_diff(q, d)
            let c: Nat satisfy { q + c = d and c != Nat.0 }
            d * m = (q + c) * m
            d * m = q * m + c * m
            q * m + (c * m + a.mod(m)) = q * m + r
            add_cancels_left(q * m, c * m + a.mod(m), r)
            c * m + a.mod(m) = r
            lte_mul(m, c)
            m <= c * m
            c * m <= r
            lte_trans(m, c * m, r)
            m <= r
            lte_imp_not_lt(m, r)
            false
        }
        trichotomy(d, q)
        d = q
        a.div(m) = q
    }
}

/// A remainder is fixed by any decomposition with a remainder below the
/// modulus.
lemma nat_mod_of_decomp_eq(a: Nat, q: Nat, r: Nat, m: Nat) {
    r < m and a = q * m + r implies a.mod(m) = r
} by {
    if r < m and a = q * m + r {
        nat_div_of_decomp_eq(a, q, r, m)
        a.div(m) = q
        div_mod_decomp(a, m)
        q * m + a.mod(m) = q * m + r
        add_cancels_left(q * m, a.mod(m), r)
        a.mod(m) = r
    }
}

/// Doubling lies below a number with quotient `q` and remainder below two
/// exactly when the undoubled value lies below `q`.
lemma double_lte_decomp_iff_lte_quotient(
    h: Nat, q: Nat, r: Nat, x: Nat
) {
    r < Nat.2 and h = q * Nat.2 + r implies
        ((Nat.2 * x <= h) = (x <= q))
} by {
    if r < Nat.2 and h = q * Nat.2 + r {
        q * Nat.2 = Nat.2 * q
        if x <= q {
            lte_mul_both(Nat.2, x, q)
            Nat.2 * x <= Nat.2 * q
            sum_lte(Nat.2 * q, Nat.0, Nat.2 * q, r)
            Nat.2 * q <= Nat.2 * q + r
            lte_trans(Nat.2 * x, Nat.2 * q, Nat.2 * q + r)
            Nat.2 * x <= h
        } else {
            q < x
            lt_imp_lte_suc(q, x)
            q.suc <= x
            lte_mul_both(Nat.2, q.suc, x)
            Nat.2 * q.suc <= Nat.2 * x
            Nat.2 * q.suc = Nat.2 * q + Nat.2
            Nat.2 = Nat.1.suc
            lt_suc_right(r, Nat.1)
            if r = Nat.1 {
                h = Nat.2 * q + Nat.1
                lt_add_suc(Nat.2 * q, Nat.1)
                Nat.2 * q + Nat.1 < Nat.2 * q + Nat.2
                h < Nat.2 * q.suc
            } else {
                r < Nat.1
                Nat.1 = Nat.0.suc
                lt_suc_right(r, Nat.0)
                not_lt_zero(r)
                r = Nat.0
                h = Nat.2 * q
                lt_add_suc(Nat.2 * q, Nat.1)
                Nat.2 * q < Nat.2 * q + Nat.2
                h < Nat.2 * q.suc
            }
            lt_and_lte(h, Nat.2 * q.suc, Nat.2 * x)
            h < Nat.2 * x
            not Nat.2 * x <= h
        }
        (Nat.2 * x <= h) = (x <= q)
    }
}

/// Multiplication by the next doubled positive integer advances both the
/// power of two and the factorial.
lemma doubling_power_factorial_step(j: Nat) {
    (Nat.2.pow(j) * j.factorial) * (Nat.2 * j.suc) =
        Nat.2.pow(j.suc) * j.suc.factorial
} by {
    exp_add(Nat.2, j, Nat.1)
    exp_one(Nat.2)
    Nat.2.pow(j.suc) = Nat.2.pow(j) * Nat.2
    factorial_step(j)
    j.suc.factorial = j.suc * j.factorial
    (Nat.2.pow(j) * j.factorial) * (Nat.2 * j.suc) =
        (Nat.2.pow(j) * Nat.2) * (j.suc * j.factorial)
    (Nat.2.pow(j) * j.factorial) * (Nat.2 * j.suc) =
        Nat.2.pow(j.suc) * j.suc.factorial
}

/// Multiplication by a predecessor and the next doubled integer advances the
/// predecessor exponent together with the doubled factorial product.
lemma pred_doubling_power_factorial_step(p: Nat, n: Nat, j: Nat) {
    ((p - Nat.1).pow(n) * (Nat.2.pow(j) * j.factorial)) *
        ((p - Nat.1) * (Nat.2 * j.suc)) =
        (p - Nat.1).pow(n.suc) *
            (Nat.2.pow(j.suc) * j.suc.factorial)
} by {
    doubling_power_factorial_step(j)
    exp_add(p - Nat.1, n, Nat.1)
    exp_one(p - Nat.1)
    (p - Nat.1).pow(n.suc) =
        (p - Nat.1).pow(n) * (p - Nat.1)
    let a = (p - Nat.1).pow(n)
    let b = Nat.2.pow(j) * j.factorial
    let c = p - Nat.1
    let d = Nat.2 * j.suc
    nat_mul_swap_middle(a, b, c, d)
    (a * b) * (c * d) = (a * c) * (b * d)
    b * d = Nat.2.pow(j.suc) * j.suc.factorial
    ((p - Nat.1).pow(n) * (Nat.2.pow(j) * j.factorial)) *
        ((p - Nat.1) * (Nat.2 * j.suc)) =
        (p - Nat.1).pow(n.suc) *
            (Nat.2.pow(j.suc) * j.suc.factorial)
}

/// Every signed doubling prefix contributes one predecessor factor for each
/// index above the quotient by two.
theorem doubling_signed_product_prefix_congr(
    p: Nat, h: Nat, q: Nat, r: Nat, k: Nat
) {
    p = Nat.2 * h + Nat.1 and r < Nat.2 and h = q * Nat.2 + r and
        k <= h implies
        doubling_signed_product_prefix(h, k).congr_mod(
            (p - Nat.1).pow(k - q) *
                (Nat.2.pow(k) * k.factorial), p)
} by {
    define pred(j: Nat) -> Bool {
        j <= h implies
            doubling_signed_product_prefix(h, j).congr_mod(
                (p - Nat.1).pow(j - q) *
                    (Nat.2.pow(j) * j.factorial), p)
    }
    if p = Nat.2 * h + Nat.1 and r < Nat.2 and h = q * Nat.2 + r {
        Nat.0.range = List.nil[Nat]
        map(List.nil[Nat], doubling_signed_index(h)) = List.nil[Nat]
        map(List.nil[Nat], Nat.suc) = List.nil[Nat]
        doubling_signed_product_prefix(h, Nat.0) = Nat.1
        if q = Nat.0 {
            sub_zero(Nat.0)
            Nat.0 - q = Nat.0
        } else {
            Nat.0 < q
            sub_lt(Nat.0, q)
            Nat.0 - q = Nat.0
        }
        (p - Nat.1).pow(Nat.0) = Nat.1
        Nat.2.pow(Nat.0) = Nat.1
        Nat.0.factorial = Nat.1
        congr_mod_refl(Nat.1, p)
        pred(Nat.0)

        forall(j: Nat) {
            if pred(j) {
                if j.suc <= h {
                    j <= j.suc
                    lte_trans(j, j.suc, h)
                    j <= h
                    lt_suc(j)
                    lt_and_lte(j, j.suc, h)
                    j < h
                    doubling_signed_product_prefix(h, j).congr_mod(
                        (p - Nat.1).pow(j - q) *
                            (Nat.2.pow(j) * j.factorial), p)
                    doubling_signed_product_prefix_suc(h, j)
                    doubling_signed_index_suc_factor_congr(p, h, j)
                    let factor = if Nat.2 * j.suc <= h {
                        Nat.2 * j.suc
                    } else {
                        (p - Nat.1) * (Nat.2 * j.suc)
                    }
                    factor = if Nat.2 * j.suc <= h {
                        Nat.2 * j.suc
                    } else {
                        (p - Nat.1) * (Nat.2 * j.suc)
                    }
                    doubling_signed_index(h)(j).suc.congr_mod(
                        if Nat.2 * j.suc <= h {
                            Nat.2 * j.suc
                        } else {
                            (p - Nat.1) * (Nat.2 * j.suc)
                        }, p)
                    doubling_signed_index(h)(j).suc.congr_mod(factor, p)
                    congr_mod_mul(
                        doubling_signed_product_prefix(h, j),
                        doubling_signed_index(h)(j).suc,
                        (p - Nat.1).pow(j - q) *
                            (Nat.2.pow(j) * j.factorial),
                        factor, p)
                    doubling_signed_product_prefix(h, j.suc).congr_mod(
                        ((p - Nat.1).pow(j - q) *
                            (Nat.2.pow(j) * j.factorial)) * factor, p)
                    double_lte_decomp_iff_lte_quotient(h, q, r, j.suc)
                    (Nat.2 * j.suc <= h) = (j.suc <= q)
                    exp_add(Nat.2, j, Nat.1)
                    exp_one(Nat.2)
                    Nat.2.pow(j.suc) = Nat.2.pow(j) * Nat.2
                    factorial_step(j)
                    j.suc.factorial = j.suc * j.factorial
                    if j.suc <= q {
                        lt_suc(j)
                        lt_and_lte(j, j.suc, q)
                        j < q
                        sub_lt(j, q)
                        j - q = Nat.0
                        if j.suc = q {
                            sub_self(q)
                            j.suc - q = Nat.0
                        } else {
                            trichotomy(j.suc, q)
                            if q < j.suc {
                                lte_imp_not_lt(j.suc, q)
                                false
                            }
                            j.suc < q
                            sub_lt(j.suc, q)
                            j.suc - q = Nat.0
                        }
                        factor = Nat.2 * j.suc
                        (p - Nat.1).pow(j - q) = Nat.1
                        (p - Nat.1).pow(j.suc - q) = Nat.1
                        doubling_power_factorial_step(j)
                        ((p - Nat.1).pow(j - q) *
                            (Nat.2.pow(j) * j.factorial)) * factor =
                            (p - Nat.1).pow(j.suc - q) *
                                (Nat.2.pow(j.suc) * j.suc.factorial)
                    } else {
                        q < j.suc
                        lt_suc_right(q, j)
                        if q = j {
                            q <= j
                        } else {
                            q < j
                            q <= j
                        }
                        add_sub(j, q)
                        j - q + q = j
                        add_sub(j.suc, q)
                        j.suc - q + q = j.suc
                        add_one_right(j - q)
                        (j - q).suc = j - q + Nat.1
                        (j - q).suc + q = j.suc
                        add_imp_sub((j - q).suc, q, j.suc)
                        j.suc - q = (j - q).suc
                        exp_add(p - Nat.1, j - q, Nat.1)
                        exp_one(p - Nat.1)
                        (p - Nat.1).pow(j.suc - q) =
                            (p - Nat.1).pow(j - q) * (p - Nat.1)
                        factor = (p - Nat.1) * (Nat.2 * j.suc)
                        pred_doubling_power_factorial_step(
                            p, j - q, j)
                        ((p - Nat.1).pow(j - q) *
                            (Nat.2.pow(j) * j.factorial)) * factor =
                            (p - Nat.1).pow(j.suc - q) *
                                (Nat.2.pow(j.suc) * j.suc.factorial)
                    }
                    doubling_signed_product_prefix(h, j.suc).congr_mod(
                        (p - Nat.1).pow(j.suc - q) *
                            (Nat.2.pow(j.suc) * j.suc.factorial), p)
                }
                pred(j.suc)
            }
        }
        forall(j: Nat) { pred(j) implies pred(j.suc) }
        pred(Nat.0) and forall(j: Nat) { pred(j) implies pred(j.suc) }
        alt_induction(pred)
        forall(j: Nat) { pred(j) }
        pred(k)
        if k <= h {
            doubling_signed_product_prefix(h, k).congr_mod(
                (p - Nat.1).pow(k - q) *
                    (Nat.2.pow(k) * k.factorial), p)
        }
    }
}

/// The number of positive integers at most `h` whose doubles lie above `h`.
define doubling_gauss_count(h: Nat) -> Nat {
    h - h.div(Nat.2)
}

/// The lower-half factorial is congruent to the product of all doubles, with
/// one predecessor factor for each double in the upper half.
theorem doubling_signed_product_congr(p: Nat, h: Nat) {
    p = Nat.2 * h + Nat.1 implies
        h.factorial.congr_mod(
            (p - Nat.1).pow(doubling_gauss_count(h)) *
                (Nat.2.pow(h) * h.factorial), p)
} by {
    if p = Nat.2 * h + Nat.1 {
        let q = h.div(Nat.2)
        let r = h.mod(Nat.2)
        div_mod_decomp(h, Nat.2)
        q * Nat.2 + r = h
        alt_suc_ne_zero(Nat.1)
        Nat.2 != Nat.0
        mod_lt(h, Nat.2)
        r < Nat.2
        doubling_signed_product_prefix_congr(p, h, q, r, h)
        doubling_signed_product_prefix(h, h).congr_mod(
            (p - Nat.1).pow(h - q) *
                (Nat.2.pow(h) * h.factorial), p)
        doubling_signed_product_prefix(h, h) = doubling_signed_product(h)
        doubling_signed_product_eq_factorial(h)
        doubling_signed_product(h) = h.factorial
        doubling_gauss_count(h) = h - q
        h.factorial.congr_mod(
            (p - Nat.1).pow(doubling_gauss_count(h)) *
                (Nat.2.pow(h) * h.factorial), p)
    }
}

/// A natural number below four is zero, one, two, or three.
lemma nat_lt_four_cases(r: Nat) {
    r < Nat.4 implies
        r = Nat.0 or r = Nat.1 or r = Nat.2 or r = Nat.3
} by {
    if r < Nat.4 {
        if r = Nat.0 {
        } else {
            if r = Nat.1 {
            } else {
                if r = Nat.2 {
                } else {
                    lt_suc_right(r, Nat.3)
                    if r != Nat.3 {
                        r < Nat.3
                        lt_suc_right(r, Nat.2)
                        r < Nat.2
                        lt_suc_right(r, Nat.1)
                        r < Nat.1
                        lt_suc_right(r, Nat.0)
                        not_lt_zero(r)
                        false
                    }
                    r = Nat.3
                }
            }
        }
        r = Nat.0 or r = Nat.1 or r = Nat.2 or r = Nat.3
    }
}

/// Twice a natural number plus one is not divisible by two.
lemma two_not_divides_double_add_one(q: Nat) {
    not Nat.2.divides(Nat.2 * q + Nat.1)
} by {
    Nat.2 * q = q * Nat.2
    Nat.2 * q + Nat.1 = q * Nat.2 + Nat.1
    Nat.1 < Nat.2
    nat_mod_of_decomp_eq(Nat.2 * q + Nat.1, q, Nat.1, Nat.2)
    (Nat.2 * q + Nat.1).mod(Nat.2) = Nat.1
    if Nat.2.divides(Nat.2 * q + Nat.1) {
        div_imp_mod(Nat.2 * q + Nat.1, Nat.2)
        (Nat.2 * q + Nat.1).mod(Nat.2) = Nat.0
        false
    }
}

/// At a multiple of four, the doubling Gauss count is twice the quotient.
lemma doubling_gauss_count_four_mul(q: Nat) {
    doubling_gauss_count(q * Nat.4) = Nat.2 * q
} by {
    Nat.4 = Nat.2 + Nat.2
    distrib_left(q, Nat.2, Nat.2)
    q * Nat.4 = q * Nat.2 + q * Nat.2
    q * Nat.2 = Nat.2 * q
    q * Nat.4 = Nat.2 * q + Nat.2 * q
    q * Nat.4 = (Nat.2 * q) * Nat.2 + Nat.0
    Nat.0 < Nat.2
    nat_div_of_decomp_eq(q * Nat.4, Nat.2 * q, Nat.0, Nat.2)
    (q * Nat.4).div(Nat.2) = Nat.2 * q
    Nat.2 * q + Nat.2 * q = q * Nat.4
    add_imp_sub(Nat.2 * q, Nat.2 * q, q * Nat.4)
    q * Nat.4 - Nat.2 * q = Nat.2 * q
    doubling_gauss_count(q * Nat.4) = Nat.2 * q
}

/// One above a multiple of four has an odd doubling Gauss count.
lemma doubling_gauss_count_four_mul_add_one(q: Nat) {
    doubling_gauss_count(q * Nat.4 + Nat.1) = Nat.2 * q + Nat.1
} by {
    Nat.4 = Nat.2 + Nat.2
    distrib_left(q, Nat.2, Nat.2)
    q * Nat.4 = q * Nat.2 + q * Nat.2
    q * Nat.2 = Nat.2 * q
    q * Nat.4 = Nat.2 * q + Nat.2 * q
    q * Nat.4 + Nat.1 = (Nat.2 * q) * Nat.2 + Nat.1
    Nat.1 < Nat.2
    nat_div_of_decomp_eq(q * Nat.4 + Nat.1, Nat.2 * q, Nat.1, Nat.2)
    (q * Nat.4 + Nat.1).div(Nat.2) = Nat.2 * q
    Nat.2 * q + Nat.1 + Nat.2 * q =
        Nat.2 * q + Nat.2 * q + Nat.1
    (Nat.2 * q + Nat.1) + Nat.2 * q = q * Nat.4 + Nat.1
    add_imp_sub(Nat.2 * q + Nat.1, Nat.2 * q, q * Nat.4 + Nat.1)
    (q * Nat.4 + Nat.1) - Nat.2 * q = Nat.2 * q + Nat.1
    doubling_gauss_count(q * Nat.4 + Nat.1) = Nat.2 * q + Nat.1
}

/// Two above a multiple of four has an odd doubling Gauss count.
lemma doubling_gauss_count_four_mul_add_two(q: Nat) {
    doubling_gauss_count(q * Nat.4 + Nat.2) = Nat.2 * q + Nat.1
} by {
    Nat.4 = Nat.2 + Nat.2
    distrib_left(q, Nat.2, Nat.2)
    q * Nat.4 = q * Nat.2 + q * Nat.2
    q * Nat.2 = Nat.2 * q
    q * Nat.4 = Nat.2 * q + Nat.2 * q
    distrib_right(Nat.2 * q, Nat.1, Nat.2)
    (Nat.2 * q + Nat.1) * Nat.2 =
        (Nat.2 * q) * Nat.2 + Nat.1 * Nat.2
    (Nat.2 * q) * Nat.2 = Nat.2 * q + Nat.2 * q
    Nat.1 * Nat.2 = Nat.2
    q * Nat.4 + Nat.2 = (Nat.2 * q + Nat.1) * Nat.2 + Nat.0
    Nat.0 < Nat.2
    nat_div_of_decomp_eq(q * Nat.4 + Nat.2, Nat.2 * q + Nat.1, Nat.0, Nat.2)
    (q * Nat.4 + Nat.2).div(Nat.2) = Nat.2 * q + Nat.1
    (Nat.2 * q + Nat.1) + (Nat.2 * q + Nat.1) = q * Nat.4 + Nat.2
    add_imp_sub(Nat.2 * q + Nat.1, Nat.2 * q + Nat.1,
        q * Nat.4 + Nat.2)
    (q * Nat.4 + Nat.2) - (Nat.2 * q + Nat.1) = Nat.2 * q + Nat.1
    doubling_gauss_count(q * Nat.4 + Nat.2) = Nat.2 * q + Nat.1
}

/// Three above a multiple of four has an even doubling Gauss count.
lemma doubling_gauss_count_four_mul_add_three(q: Nat) {
    doubling_gauss_count(q * Nat.4 + Nat.3) = Nat.2 * (q + Nat.1)
} by {
    Nat.4 = Nat.2 + Nat.2
    distrib_left(q, Nat.2, Nat.2)
    q * Nat.4 = q * Nat.2 + q * Nat.2
    q * Nat.2 = Nat.2 * q
    q * Nat.4 = Nat.2 * q + Nat.2 * q
    distrib_right(Nat.2 * q, Nat.1, Nat.2)
    (Nat.2 * q + Nat.1) * Nat.2 =
        (Nat.2 * q) * Nat.2 + Nat.1 * Nat.2
    (Nat.2 * q) * Nat.2 = Nat.2 * q + Nat.2 * q
    Nat.1 * Nat.2 = Nat.2
    q * Nat.4 + Nat.3 = (Nat.2 * q + Nat.1) * Nat.2 + Nat.1
    Nat.1 < Nat.2
    nat_div_of_decomp_eq(q * Nat.4 + Nat.3, Nat.2 * q + Nat.1, Nat.1, Nat.2)
    (q * Nat.4 + Nat.3).div(Nat.2) = Nat.2 * q + Nat.1
    mul_two_left(q + Nat.1)
    Nat.2 * (q + Nat.1) = (q + Nat.1) + (q + Nat.1)
    Nat.2 * (q + Nat.1) = Nat.2 * q + Nat.2
    Nat.2 * q + Nat.2 + (Nat.2 * q + Nat.1) =
        Nat.2 * q + Nat.2 * q + Nat.3
    Nat.2 * (q + Nat.1) + (Nat.2 * q + Nat.1) = q * Nat.4 + Nat.3
    add_imp_sub(Nat.2 * (q + Nat.1), Nat.2 * q + Nat.1,
        q * Nat.4 + Nat.3)
    (q * Nat.4 + Nat.3) - (Nat.2 * q + Nat.1) = Nat.2 * (q + Nat.1)
    doubling_gauss_count(q * Nat.4 + Nat.3) = Nat.2 * (q + Nat.1)
}

/// The doubling Gauss count is even exactly when the half-size has remainder
/// zero or three modulo four.
theorem doubling_gauss_count_even_iff_mod_four_zero_or_three(h: Nat) {
    Nat.2.divides(doubling_gauss_count(h)) =
        (h.mod(Nat.4) = Nat.0 or h.mod(Nat.4) = Nat.3)
} by {
    let q = h.div(Nat.4)
    let r = h.mod(Nat.4)
    div_mod_decomp(h, Nat.4)
    q * Nat.4 + r = h
    alt_suc_ne_zero(Nat.3)
    Nat.4 != Nat.0
    mod_lt(h, Nat.4)
    r < Nat.4
    nat_lt_four_cases(r)
    r = Nat.0 or r = Nat.1 or r = Nat.2 or r = Nat.3
    Nat.2.divides(doubling_gauss_count(h)) = exists(k: Nat) {
        Nat.2 * k = doubling_gauss_count(h)
    }

    if r = Nat.0 {
        h = q * Nat.4
        doubling_gauss_count_four_mul(q)
        doubling_gauss_count(h) = Nat.2 * q
        Nat.2.divides(doubling_gauss_count(h))
        h.mod(Nat.4) = Nat.0
        Nat.2.divides(doubling_gauss_count(h)) =
            (h.mod(Nat.4) = Nat.0 or h.mod(Nat.4) = Nat.3)
    } else {
        if r = Nat.1 {
            h = q * Nat.4 + Nat.1
            doubling_gauss_count_four_mul_add_one(q)
            doubling_gauss_count(h) = Nat.2 * q + Nat.1
            two_not_divides_double_add_one(q)
            not Nat.2.divides(doubling_gauss_count(h))
            h.mod(Nat.4) != Nat.0
            h.mod(Nat.4) != Nat.3
            Nat.2.divides(doubling_gauss_count(h)) =
                (h.mod(Nat.4) = Nat.0 or h.mod(Nat.4) = Nat.3)
        } else {
            if r = Nat.2 {
                h = q * Nat.4 + Nat.2
                doubling_gauss_count_four_mul_add_two(q)
                doubling_gauss_count(h) = Nat.2 * q + Nat.1
                two_not_divides_double_add_one(q)
                not Nat.2.divides(doubling_gauss_count(h))
                h.mod(Nat.4) != Nat.0
                h.mod(Nat.4) != Nat.3
                Nat.2.divides(doubling_gauss_count(h)) =
                    (h.mod(Nat.4) = Nat.0 or h.mod(Nat.4) = Nat.3)
            } else {
                r = Nat.3
                h = q * Nat.4 + Nat.3
                doubling_gauss_count_four_mul_add_three(q)
                doubling_gauss_count(h) = Nat.2 * (q + Nat.1)
                Nat.2.divides(doubling_gauss_count(h))
                h.mod(Nat.4) = Nat.3
                Nat.2.divides(doubling_gauss_count(h)) =
                    (h.mod(Nat.4) = Nat.0 or h.mod(Nat.4) = Nat.3)
            }
        }
    }
}

/// Twice a remainder below four, plus one, lies below eight.
lemma double_remainder_add_one_lt_eight(r: Nat) {
    r < Nat.4 implies Nat.2 * r + Nat.1 < Nat.8
} by {
    if r < Nat.4 {
        lt_suc_right(r, Nat.3)
        if r = Nat.3 {
            Nat.2 * r + Nat.1 = Nat.7
            lt_suc(Nat.7)
            Nat.7.suc = Nat.8
            Nat.2 * r + Nat.1 < Nat.8
        } else {
            r < Nat.3
            alt_suc_ne_zero(Nat.1)
            Nat.2 != Nat.0
            lt_mul_both(Nat.2, r, Nat.3)
            Nat.2 * r < Nat.2 * Nat.3
            Nat.2 * Nat.3 = Nat.6
            lt_add_left(Nat.1, Nat.2 * r, Nat.6)
            Nat.1 + Nat.2 * r < Nat.1 + Nat.6
            Nat.2 * r + Nat.1 = Nat.1 + Nat.2 * r
            Nat.1 + Nat.6 = Nat.7
            Nat.2 * r + Nat.1 < Nat.7
            lt_suc(Nat.7)
            Nat.7.suc = Nat.8
            Nat.7 < Nat.8
            lt_trans(Nat.2 * r + Nat.1, Nat.7, Nat.8)
            Nat.2 * r + Nat.1 < Nat.8
        }
    }
}

/// Doubling a number and adding one carries its remainder modulo four to the
/// corresponding odd remainder modulo eight.
theorem double_add_one_mod_eight_from_mod_four(h: Nat) {
    (Nat.2 * h + Nat.1).mod(Nat.8) =
        Nat.2 * h.mod(Nat.4) + Nat.1
} by {
    let q = h.div(Nat.4)
    let r = h.mod(Nat.4)
    div_mod_decomp(h, Nat.4)
    q * Nat.4 + r = h
    alt_suc_ne_zero(Nat.3)
    Nat.4 != Nat.0
    mod_lt(h, Nat.4)
    r < Nat.4
    nat_lt_four_cases(r)
    r = Nat.0 or r = Nat.1 or r = Nat.2 or r = Nat.3
    Nat.8 = Nat.2 * Nat.4
    Nat.2 * h + Nat.1 = Nat.2 * (q * Nat.4 + r) + Nat.1
    distrib_left(Nat.2, q * Nat.4, r)
    Nat.2 * (q * Nat.4 + r) = Nat.2 * (q * Nat.4) + Nat.2 * r
    mul_assoc(Nat.2, q, Nat.4)
    Nat.2 * (q * Nat.4) = (Nat.2 * q) * Nat.4
    Nat.2 * q = q * Nat.2
    mul_assoc(q, Nat.2, Nat.4)
    (q * Nat.2) * Nat.4 = q * (Nat.2 * Nat.4)
    Nat.2 * (q * Nat.4) = q * Nat.8
    Nat.2 * h + Nat.1 = q * Nat.8 + (Nat.2 * r + Nat.1)
    double_remainder_add_one_lt_eight(r)
    Nat.2 * r + Nat.1 < Nat.8
    nat_mod_of_decomp_eq(
        Nat.2 * h + Nat.1, q, Nat.2 * r + Nat.1, Nat.8)
    (Nat.2 * h + Nat.1).mod(Nat.8) = Nat.2 * r + Nat.1
    (Nat.2 * h + Nat.1).mod(Nat.8) =
        Nat.2 * h.mod(Nat.4) + Nat.1
}

/// The doubling Gauss count for an odd number is even exactly in residue
/// classes one and seven modulo eight.
theorem doubling_gauss_count_even_iff_congr_one_or_seven_mod_eight(
    p: Nat, h: Nat
) {
    p = Nat.2 * h + Nat.1 implies
        (Nat.2.divides(doubling_gauss_count(h)) =
            (p.congr_mod(Nat.1, Nat.8) or
                p.congr_mod(Nat.7, Nat.8)))
} by {
    if p = Nat.2 * h + Nat.1 {
        doubling_gauss_count_even_iff_mod_four_zero_or_three(h)
        double_add_one_mod_eight_from_mod_four(h)
        p.mod(Nat.8) = Nat.2 * h.mod(Nat.4) + Nat.1
        lt_add_suc(Nat.1, Nat.6)
        Nat.1 + Nat.7 = Nat.8
        Nat.1 < Nat.8
        small_mod(Nat.1, Nat.8)
        Nat.1.mod(Nat.8) = Nat.1
        lt_add_suc(Nat.7, Nat.0)
        Nat.7 + Nat.1 = Nat.8
        Nat.7 < Nat.8
        small_mod(Nat.7, Nat.8)
        Nat.7.mod(Nat.8) = Nat.7
        p.congr_mod(Nat.1, Nat.8) = (p.mod(Nat.8) = Nat.1)
        p.congr_mod(Nat.7, Nat.8) = (p.mod(Nat.8) = Nat.7)
        alt_suc_ne_zero(Nat.3)
        Nat.4 != Nat.0
        mod_lt(h, Nat.4)
        h.mod(Nat.4) < Nat.4
        nat_lt_four_cases(h.mod(Nat.4))
        h.mod(Nat.4) = Nat.0 or h.mod(Nat.4) = Nat.1 or
            h.mod(Nat.4) = Nat.2 or h.mod(Nat.4) = Nat.3
        if h.mod(Nat.4) = Nat.0 {
            p.mod(Nat.8) = Nat.1
            Nat.2.divides(doubling_gauss_count(h))
            p.congr_mod(Nat.1, Nat.8)
            Nat.2.divides(doubling_gauss_count(h)) =
                (p.congr_mod(Nat.1, Nat.8) or
                    p.congr_mod(Nat.7, Nat.8))
        } else {
            if h.mod(Nat.4) = Nat.1 {
                p.mod(Nat.8) = Nat.3
                not Nat.2.divides(doubling_gauss_count(h))
                not p.congr_mod(Nat.1, Nat.8)
                not p.congr_mod(Nat.7, Nat.8)
                Nat.2.divides(doubling_gauss_count(h)) =
                    (p.congr_mod(Nat.1, Nat.8) or
                        p.congr_mod(Nat.7, Nat.8))
            } else {
                if h.mod(Nat.4) = Nat.2 {
                    p.mod(Nat.8) = Nat.5
                    not Nat.2.divides(doubling_gauss_count(h))
                    not p.congr_mod(Nat.1, Nat.8)
                    not p.congr_mod(Nat.7, Nat.8)
                    Nat.2.divides(doubling_gauss_count(h)) =
                        (p.congr_mod(Nat.1, Nat.8) or
                            p.congr_mod(Nat.7, Nat.8))
                } else {
                    h.mod(Nat.4) = Nat.3
                    p.mod(Nat.8) = Nat.7
                    Nat.2.divides(doubling_gauss_count(h))
                    p.congr_mod(Nat.7, Nat.8)
                    Nat.2.divides(doubling_gauss_count(h)) =
                        (p.congr_mod(Nat.1, Nat.8) or
                            p.congr_mod(Nat.7, Nat.8))
                }
            }
        }
    }
}

/// The lower-half factorial is coprime to its odd prime modulus.
theorem prime_half_factorial_coprime(p: Nat, h: Nat) {
    p.is_prime and p = Nat.2 * h + Nat.1 implies h.factorial.coprime(p)
} by {
    if p.is_prime and p = Nat.2 * h + Nat.1 {
        lte_mul(h, Nat.2)
        h <= h * Nat.2
        h * Nat.2 = Nat.2 * h
        h <= Nat.2 * h
        lt_suc(Nat.2 * h)
        Nat.2 * h < p
        lte_and_lt(h, Nat.2 * h, p)
        h < p
        prime_does_not_divide_factorial(p, h)
        not p.divides(h.factorial)
        if not h.factorial.coprime(p) {
            not_coprime_imp_divides_prime(p, h.factorial)
            p.divides(h.factorial)
            false
        }
        h.factorial.coprime(p)
    }
}

/// Gauss's signed doubling product determines the half-power of two up to
/// the predecessor sign.
theorem doubling_sign_times_half_power_congr_one(p: Nat, h: Nat) {
    p.is_prime and p = Nat.2 * h + Nat.1 implies
        ((p - Nat.1).pow(doubling_gauss_count(h)) * Nat.2.pow(h)).congr_mod(
            Nat.1, p)
} by {
    if p.is_prime and p = Nat.2 * h + Nat.1 {
        doubling_signed_product_congr(p, h)
        h.factorial.congr_mod(
            (p - Nat.1).pow(doubling_gauss_count(h)) *
                (Nat.2.pow(h) * h.factorial), p)
        h.factorial * Nat.1 = h.factorial
        (p - Nat.1).pow(doubling_gauss_count(h)) *
                (Nat.2.pow(h) * h.factorial) =
            h.factorial *
                ((p - Nat.1).pow(doubling_gauss_count(h)) * Nat.2.pow(h))
        (h.factorial * Nat.1).congr_mod(
            h.factorial *
                ((p - Nat.1).pow(doubling_gauss_count(h)) * Nat.2.pow(h)), p)
        prime_half_factorial_coprime(p, h)
        h.factorial.coprime(p)
        cancel_coprime(
            h.factorial, p, Nat.1,
            (p - Nat.1).pow(doubling_gauss_count(h)) * Nat.2.pow(h))
        Nat.1.congr_mod(
            (p - Nat.1).pow(doubling_gauss_count(h)) * Nat.2.pow(h), p)
        congr_mod_symm(
            Nat.1,
            (p - Nat.1).pow(doubling_gauss_count(h)) * Nat.2.pow(h), p)
        ((p - Nat.1).pow(doubling_gauss_count(h)) * Nat.2.pow(h)).congr_mod(
            Nat.1, p)
    }
}

/// The half-power of two is one modulo an odd prime exactly when the Gauss
/// doubling count is even.
theorem two_half_power_congr_one_iff_doubling_gauss_count_even(
    p: Nat, h: Nat
) {
    p.is_prime and p = Nat.2 * h + Nat.1 implies
        (Nat.2.pow(h).congr_mod(Nat.1, p) =
            Nat.2.divides(doubling_gauss_count(h)))
} by {
    if p.is_prime and p = Nat.2 * h + Nat.1 {
        if h = Nat.0 {
            p = Nat.1
            Nat.1 < p
            Nat.1 < Nat.1
            lt_not_ref(Nat.1)
            false
        }
        h != Nat.0
        alt_suc_ne_zero(Nat.1)
        Nat.2 != Nat.0
        lte_mul(Nat.2, h)
        Nat.2 <= Nat.2 * h
        lt_suc(Nat.2 * h)
        Nat.2 * h < p
        lte_and_lt(Nat.2, Nat.2 * h, p)
        Nat.2 < p
        doubling_sign_times_half_power_congr_one(p, h)
        ((p - Nat.1).pow(doubling_gauss_count(h)) * Nat.2.pow(h)).congr_mod(
            Nat.1, p)
        if Nat.2.divides(doubling_gauss_count(h)) {
            let q: Nat satisfy {
                Nat.2 * q = doubling_gauss_count(h)
            }
            prime_pred_even_power_congr_one(p, q)
            (p - Nat.1).pow(Nat.2 * q).congr_mod(Nat.1, p)
            (p - Nat.1).pow(doubling_gauss_count(h)).congr_mod(Nat.1, p)
            congr_mod_refl(Nat.2.pow(h), p)
            congr_mod_mul(
                (p - Nat.1).pow(doubling_gauss_count(h)), Nat.2.pow(h),
                Nat.1, Nat.2.pow(h), p)
            ((p - Nat.1).pow(doubling_gauss_count(h)) * Nat.2.pow(h)).congr_mod(
                Nat.2.pow(h), p)
            congr_mod_symm(
                (p - Nat.1).pow(doubling_gauss_count(h)) * Nat.2.pow(h),
                Nat.2.pow(h), p)
            Nat.2.pow(h).congr_mod(
                (p - Nat.1).pow(doubling_gauss_count(h)) * Nat.2.pow(h), p)
            congr_mod_trans(
                Nat.2.pow(h),
                (p - Nat.1).pow(doubling_gauss_count(h)) * Nat.2.pow(h),
                Nat.1, p)
            Nat.2.pow(h).congr_mod(Nat.1, p)
        }
        if Nat.2.pow(h).congr_mod(Nat.1, p) {
            if not Nat.2.divides(doubling_gauss_count(h)) {
                odd_decomp_of_not_two_divides(doubling_gauss_count(h))
                let q: Nat satisfy {
                    doubling_gauss_count(h) = Nat.2 * q + Nat.1
                }
                prime_pred_odd_power_congr_pred(p, q)
                (p - Nat.1).pow(Nat.2 * q + Nat.1).congr_mod(p - Nat.1, p)
                (p - Nat.1).pow(doubling_gauss_count(h)).congr_mod(p - Nat.1, p)
                congr_mod_mul(
                    (p - Nat.1).pow(doubling_gauss_count(h)), Nat.2.pow(h),
                    p - Nat.1, Nat.1, p)
                ((p - Nat.1).pow(doubling_gauss_count(h)) * Nat.2.pow(h)).congr_mod(
                    (p - Nat.1) * Nat.1, p)
                (p - Nat.1) * Nat.1 = p - Nat.1
                ((p - Nat.1).pow(doubling_gauss_count(h)) * Nat.2.pow(h)).congr_mod(
                    p - Nat.1, p)
                congr_mod_symm(
                    (p - Nat.1).pow(doubling_gauss_count(h)) * Nat.2.pow(h),
                    Nat.1, p)
                Nat.1.congr_mod(
                    (p - Nat.1).pow(doubling_gauss_count(h)) * Nat.2.pow(h), p)
                congr_mod_trans(
                    Nat.1,
                    (p - Nat.1).pow(doubling_gauss_count(h)) * Nat.2.pow(h),
                    p - Nat.1, p)
                Nat.1.congr_mod(p - Nat.1, p)
                Nat.1 < p
                small_mod(Nat.1, p)
                Nat.1.mod(p) = Nat.1
                sub_one_lt(p)
                p - Nat.1 < p
                small_mod(p - Nat.1, p)
                (p - Nat.1).mod(p) = p - Nat.1
                Nat.1.congr_mod(p - Nat.1, p) =
                    (Nat.1.mod(p) = (p - Nat.1).mod(p))
                Nat.1.mod(p) = (p - Nat.1).mod(p)
                Nat.1 = p - Nat.1
                false
            }
            Nat.2.divides(doubling_gauss_count(h))
        }
        (Nat.2.pow(h).congr_mod(Nat.1, p) =
            Nat.2.divides(doubling_gauss_count(h))) = true
    }
}

/// Two is coprime to an odd prime written as `2*h + 1`.
theorem odd_prime_two_coprime(p: Nat, h: Nat) {
    p.is_prime and p = Nat.2 * h + Nat.1 implies Nat.2.coprime(p)
} by {
    if p.is_prime and p = Nat.2 * h + Nat.1 {
        if h = Nat.0 {
            p = Nat.1
            Nat.1 < p
            Nat.1 < Nat.1
            lt_not_ref(Nat.1)
            false
        }
        h != Nat.0
        alt_suc_ne_zero(Nat.1)
        Nat.2 != Nat.0
        lte_mul(Nat.2, h)
        Nat.2 <= Nat.2 * h
        lt_suc(Nat.2 * h)
        Nat.2 * h < p
        lte_and_lt(Nat.2, Nat.2 * h, p)
        Nat.2 < p
        Nat.1 <= Nat.2
        coprime_below_prime(p, Nat.2)
        Nat.2.coprime(p)
    }
}

/// Two is a quadratic residue modulo an odd prime exactly in residue classes
/// one and seven modulo eight, assuming existence of a full unit generator.
theorem prime_two_quadratic_residue_iff_congr_one_or_seven_mod_eight(
    p: Nat, h: Nat
) {
    p.is_prime and p = Nat.2 * h + Nat.1 and
        (exists(g: Nat) { is_order_double_unit_generator_mod(g, p, h) })
    implies (is_quadratic_residue_mod(Nat.2, p) =
        (p.congr_mod(Nat.1, Nat.8) or p.congr_mod(Nat.7, Nat.8)))
} by {
    if p.is_prime and p = Nat.2 * h + Nat.1 and
        (exists(g: Nat) { is_order_double_unit_generator_mod(g, p, h) }) {
        odd_prime_two_coprime(p, h)
        Nat.2.coprime(p)
        existing_order_double_unit_generator_euler_criterion_iff(
            p, h, Nat.2)
        is_quadratic_residue_mod(Nat.2, p) =
            Nat.2.pow(h).congr_mod(Nat.1, p)
        two_half_power_congr_one_iff_doubling_gauss_count_even(p, h)
        Nat.2.pow(h).congr_mod(Nat.1, p) =
            Nat.2.divides(doubling_gauss_count(h))
        doubling_gauss_count_even_iff_congr_one_or_seven_mod_eight(p, h)
        Nat.2.divides(doubling_gauss_count(h)) =
            (p.congr_mod(Nat.1, Nat.8) or p.congr_mod(Nat.7, Nat.8))
        is_quadratic_residue_mod(Nat.2, p) =
            (p.congr_mod(Nat.1, Nat.8) or p.congr_mod(Nat.7, Nat.8))
    }
}

/// The Legendre symbol of two is one in residue classes one and seven modulo
/// eight and minus one in the other odd residue classes, assuming existence
/// of a full unit generator.
theorem prime_two_legendre_symbol_mod_eight(p: Nat, h: Nat) {
    p.is_prime and p = Nat.2 * h + Nat.1 and
        (exists(g: Nat) { is_order_double_unit_generator_mod(g, p, h) })
    implies legendre_symbol(Nat.2, p) =
        if p.congr_mod(Nat.1, Nat.8) or p.congr_mod(Nat.7, Nat.8) {
            Int.1
        } else {
            -Int.1
        }
} by {
    if p.is_prime and p = Nat.2 * h + Nat.1 and
        (exists(g: Nat) { is_order_double_unit_generator_mod(g, p, h) }) {
        odd_prime_two_coprime(p, h)
        Nat.2.coprime(p)
        prime_two_quadratic_residue_iff_congr_one_or_seven_mod_eight(p, h)
        is_quadratic_residue_mod(Nat.2, p) =
            (p.congr_mod(Nat.1, Nat.8) or p.congr_mod(Nat.7, Nat.8))
        if is_quadratic_residue_mod(Nat.2, p) {
            quadratic_residue_coprime_is_unit(Nat.2, p)
            is_unit_quadratic_residue_mod(Nat.2, p)
        }
        if is_unit_quadratic_residue_mod(Nat.2, p) {
            unit_quadratic_residue_is_residue(Nat.2, p)
            is_quadratic_residue_mod(Nat.2, p)
        }
        is_unit_quadratic_residue_mod(Nat.2, p) =
            is_quadratic_residue_mod(Nat.2, p)
        legendre_symbol_value_one_iff_prime_unit_quadratic_residue(p, Nat.2)
        (legendre_symbol(Nat.2, p) = Int.1) =
            is_unit_quadratic_residue_mod(Nat.2, p)
        (legendre_symbol(Nat.2, p) = Int.1) =
            (p.congr_mod(Nat.1, Nat.8) or p.congr_mod(Nat.7, Nat.8))
        legendre_symbol_prime_coprime_unit_value_cases(p, Nat.2)
        legendre_symbol(Nat.2, p) = Int.1 or
            legendre_symbol(Nat.2, p) = -Int.1
        if p.congr_mod(Nat.1, Nat.8) or p.congr_mod(Nat.7, Nat.8) {
            legendre_symbol(Nat.2, p) = Int.1
            legendre_symbol(Nat.2, p) =
                if p.congr_mod(Nat.1, Nat.8) or p.congr_mod(Nat.7, Nat.8) {
                    Int.1
                } else {
                    -Int.1
                }
        } else {
            legendre_symbol(Nat.2, p) != Int.1
            legendre_symbol(Nat.2, p) = -Int.1
            legendre_symbol(Nat.2, p) =
                if p.congr_mod(Nat.1, Nat.8) or p.congr_mod(Nat.7, Nat.8) {
                    Int.1
                } else {
                    -Int.1
                }
        }
    }
}
