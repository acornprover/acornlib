/// Periodic continued fractions.
///
/// The continued fractions of the quadratic irrationals √2 and √3 are
/// periodic:
///
///     √2 = [1; 2, 2, 2, ...]   and   √3 = [1; 1, 2, 1, 2, ...].
///
/// This file formalizes the coefficient sequences of these two examples and
/// their first convergents:
///
///   1. The continued fraction of √2 is [1; 2, 2, 2, ...]: the integral part
///      is one and every later partial quotient is two (Section 1).  The
///      coefficient sequence is the one defined in pell.ac and its properties
///      are restated here for reference.
///
///   2. The continued fraction of √3 is [1; 1, 2, 1, 2, ...]: the first few
///      partial quotients are 1, 1, 2, 1, 2, and in general the odd partial
///      quotients are one and the even ones (from the second on) are two
///      (Section 2).  The first convergents are 1/1, 2/1, 5/3, 7/4 (Section 3).
///
///   3. The convergents of √2 have Pell norm one or minus one,
///      p_n² - 2·q_n² = ±1, alternating with period two; this is the classical
///      identity proved in cf_pell.ac, restated here (Section 4).  The
///      corresponding norm pattern for the convergents of √3 is computed for
///      the first four convergents, and the general statement is left as a
///      stated theorem.
///
///   4. Lagrange's theorem — the continued fraction of a quadratic irrational
///      is eventually periodic — and its converse are stated at the end
///      (Section 5), together with the definition of eventual periodicity and
///      the proof that the coefficient sequence of √2 is eventually periodic.
///      The converse is classical and elementary; the forward direction is the
///      deep theorem.  Both are left for future work.
from nat import Nat, mod_of_zero, small_mod, mod_of_decomp, mul_comm,
    add_zero_right, add_one_right, add_suc_right, add_assoc, add_comm, mul_suc_right
from int import Int, abs, mul_from_nat, add_from_nat, mul_assoc,
    mul_neg_left, neg_sub, sub_self, sub_nat, sub_nat_add_left, neg_sub_nat,
    mul_one_left, mul_one_right
from rat import Rat
from real import Real
from pair import Pair, pair_new_second
from number_theory.continued_fraction_convergents import continued_fraction_convergent_numerator,
    continued_fraction_convergent_denominator, continued_fraction_convergent_value,
    continued_fraction_convergent_numerator_zero, continued_fraction_convergent_denominator_zero,
    continued_fraction_convergent_numerator_suc, continued_fraction_convergent_denominator_suc,
    continued_fraction_recurrence_state, continued_fraction_recurrence_state_suc_first,
    continued_fraction_recurrence_state_zero, positive_continued_fraction_sequence_tail
from number_theory.continued_fraction_approx import continued_fraction_convergent_numerator_two_suc,
    continued_fraction_convergent_denominator_two_suc, continued_fraction_real_limit
from number_theory.pell import sqrt_two_continued_fraction_coefficients, pell_norm,
    sqrt_two_convergent_numerator_zero, sqrt_two_convergent_denominator_zero,
    sqrt_two_convergent_numerator_one, sqrt_two_convergent_denominator_one,
    pell_rearrange_sub_sum, pell_rearrange_flatten
from number_theory.cf_pell import cf_pell_convergent_norm_alternating,
    cf_pell_convergent_norm_one_or_neg_one, cf_pell_convergent_norm_abs_one,
    cf_pell_sub_pair_identity
from algebra.ring.ring import alternating_sign, alternating_sign_suc, mul_neg_one_left

numerals Nat
numerals Int

// ============================================================================
// Section 1: the continued fraction of √2 is [1; 2, 2, 2, ...]
// ============================================================================

/// The integral part of the continued fraction of √2 is one.
theorem continued_fraction_periodic_sqrt_two_coefficient_zero {
    sqrt_two_continued_fraction_coefficients(Nat.0) = Nat.1
} by {
    sqrt_two_continued_fraction_coefficients(Nat.0) = Nat.1
}

/// Every partial quotient after the integral part of the continued fraction
/// of √2 is two.
theorem continued_fraction_periodic_sqrt_two_coefficient_suc(n: Nat) {
    sqrt_two_continued_fraction_coefficients(n.suc) = Nat.2
} by {
    sqrt_two_continued_fraction_coefficients(n.suc) = Nat.2
}

/// The first three partial quotients of the continued fraction of √2 are
/// 1, 2, 2.
theorem continued_fraction_periodic_sqrt_two_first_partial_quotients {
    sqrt_two_continued_fraction_coefficients(Nat.0) = Nat.1 and
        sqrt_two_continued_fraction_coefficients(Nat.1) = Nat.2 and
        sqrt_two_continued_fraction_coefficients(Nat.2) = Nat.2
} by {
    continued_fraction_periodic_sqrt_two_coefficient_zero
    sqrt_two_continued_fraction_coefficients(Nat.0) = Nat.1
    continued_fraction_periodic_sqrt_two_coefficient_suc(Nat.0)
    sqrt_two_continued_fraction_coefficients(Nat.1) = Nat.2
    continued_fraction_periodic_sqrt_two_coefficient_suc(Nat.1)
    sqrt_two_continued_fraction_coefficients(Nat.2) = Nat.2
    sqrt_two_continued_fraction_coefficients(Nat.0) = Nat.1 and
        sqrt_two_continued_fraction_coefficients(Nat.1) = Nat.2 and
        sqrt_two_continued_fraction_coefficients(Nat.2) = Nat.2
}

/// The partial quotients of √2 repeat with period one after the integral
/// part: every partial quotient equals the next one.
theorem continued_fraction_periodic_sqrt_two_period_one(n: Nat) {
    sqrt_two_continued_fraction_coefficients(n.suc) =
        sqrt_two_continued_fraction_coefficients(n.suc.suc)
} by {
    continued_fraction_periodic_sqrt_two_coefficient_suc(n)
    sqrt_two_continued_fraction_coefficients(n.suc) = Nat.2
    continued_fraction_periodic_sqrt_two_coefficient_suc(n.suc)
    sqrt_two_continued_fraction_coefficients(n.suc.suc) = Nat.2
    sqrt_two_continued_fraction_coefficients(n.suc) =
        sqrt_two_continued_fraction_coefficients(n.suc.suc)
}

/// Every coefficient of the continued fraction of √2 after the integral part
/// is positive, so the coefficient sequence has a positive tail.
theorem continued_fraction_periodic_sqrt_two_positive_tail {
    positive_continued_fraction_sequence_tail(sqrt_two_continued_fraction_coefficients)
} by {
    forall(n: Nat) {
        continued_fraction_periodic_sqrt_two_coefficient_suc(n)
        sqrt_two_continued_fraction_coefficients(n.suc) = Nat.2
        Nat.0 < Nat.2
        Nat.0 < sqrt_two_continued_fraction_coefficients(n.suc)
    }
    positive_continued_fraction_sequence_tail(sqrt_two_continued_fraction_coefficients)
}

/// The first convergents of √2 are 1/1, 3/2, 7/5: the numerators are 1, 3, 7
/// and the denominators are 1, 2, 5.
theorem continued_fraction_periodic_sqrt_two_first_convergents {
    continued_fraction_convergent_numerator(
            sqrt_two_continued_fraction_coefficients, Nat.0) = Nat.1 and
    continued_fraction_convergent_denominator(
            sqrt_two_continued_fraction_coefficients, Nat.0) = Nat.1 and
    continued_fraction_convergent_numerator(
            sqrt_two_continued_fraction_coefficients, Nat.1) = Nat.3 and
    continued_fraction_convergent_denominator(
            sqrt_two_continued_fraction_coefficients, Nat.1) = Nat.2
} by {
    sqrt_two_convergent_numerator_zero
    continued_fraction_convergent_numerator(
        sqrt_two_continued_fraction_coefficients, Nat.0) = Nat.1
    sqrt_two_convergent_denominator_zero
    continued_fraction_convergent_denominator(
        sqrt_two_continued_fraction_coefficients, Nat.0) = Nat.1
    sqrt_two_convergent_numerator_one
    continued_fraction_convergent_numerator(
        sqrt_two_continued_fraction_coefficients, Nat.1) = Nat.3
    sqrt_two_convergent_denominator_one
    continued_fraction_convergent_denominator(
        sqrt_two_continued_fraction_coefficients, Nat.1) = Nat.2
    continued_fraction_convergent_numerator(
            sqrt_two_continued_fraction_coefficients, Nat.0) = Nat.1 and
    continued_fraction_convergent_denominator(
            sqrt_two_continued_fraction_coefficients, Nat.0) = Nat.1 and
    continued_fraction_convergent_numerator(
            sqrt_two_continued_fraction_coefficients, Nat.1) = Nat.3 and
    continued_fraction_convergent_denominator(
            sqrt_two_continued_fraction_coefficients, Nat.1) = Nat.2
}

// ============================================================================
// Section 2: the continued fraction of √3 is [1; 1, 2, 1, 2, ...]
// ============================================================================

/// The coefficients of the continued fraction of √3: [1; 1, 2, 1, 2, ...].
///
/// The integral part is one; after it, the coefficient at an odd index is
/// one and the coefficient at an even index is two.
define sqrt_three_continued_fraction_coefficients(n: Nat) -> Nat {
    match n {
        Nat.zero {
            Nat.1
        }
        Nat.suc(m) {
            if m.mod(Nat.2) = Nat.0 {
                Nat.1
            } else {
                Nat.2
            }
        }
    }
}

/// The integral part of the continued fraction of √3 is one.
theorem continued_fraction_periodic_sqrt_three_coefficient_zero {
    sqrt_three_continued_fraction_coefficients(Nat.0) = Nat.1
} by {
    sqrt_three_continued_fraction_coefficients(Nat.0) = Nat.1
}

/// The first partial quotient of √3 is one.
theorem continued_fraction_periodic_sqrt_three_coefficient_one {
    sqrt_three_continued_fraction_coefficients(Nat.1) = Nat.1
} by {
    mod_of_zero(Nat.2)
    Nat.0.mod(Nat.2) = Nat.0
    sqrt_three_continued_fraction_coefficients(Nat.1) = Nat.1
}

/// The second partial quotient of √3 is two.
theorem continued_fraction_periodic_sqrt_three_coefficient_two {
    sqrt_three_continued_fraction_coefficients(Nat.2) = Nat.2
} by {
    Nat.1 < Nat.2
    small_mod(Nat.1, Nat.2)
    Nat.1.mod(Nat.2) = Nat.1
    Nat.1.mod(Nat.2) != Nat.0
    sqrt_three_continued_fraction_coefficients(Nat.2) = Nat.2
}

/// The third partial quotient of √3 is one.
theorem continued_fraction_periodic_sqrt_three_coefficient_three {
    sqrt_three_continued_fraction_coefficients(Nat.3) = Nat.1
} by {
    Nat.0 < Nat.2
    mod_of_decomp(Nat.1, Nat.0, Nat.2)
    (Nat.1 * Nat.2 + Nat.0).mod(Nat.2) = Nat.0
    Nat.1 * Nat.2 = Nat.2
    add_zero_right(Nat.2)
    Nat.2 + Nat.0 = Nat.2
    Nat.2.mod(Nat.2) = Nat.0
    sqrt_three_continued_fraction_coefficients(Nat.3) = Nat.1
}

/// The fourth partial quotient of √3 is two.
theorem continued_fraction_periodic_sqrt_three_coefficient_four {
    sqrt_three_continued_fraction_coefficients(Nat.4) = Nat.2
} by {
    Nat.1 < Nat.2
    mod_of_decomp(Nat.1, Nat.1, Nat.2)
    (Nat.1 * Nat.2 + Nat.1).mod(Nat.2) = Nat.1
    Nat.1 * Nat.2 = Nat.2
    Nat.2 + Nat.1 = Nat.3
    Nat.3.mod(Nat.2) = Nat.1
    Nat.3.mod(Nat.2) != Nat.0
    sqrt_three_continued_fraction_coefficients(Nat.4) = Nat.2
}

/// The first five partial quotients of the continued fraction of √3 are
/// 1, 1, 2, 1, 2.
theorem continued_fraction_periodic_sqrt_three_first_partial_quotients {
    sqrt_three_continued_fraction_coefficients(Nat.0) = Nat.1 and
        sqrt_three_continued_fraction_coefficients(Nat.1) = Nat.1 and
        sqrt_three_continued_fraction_coefficients(Nat.2) = Nat.2 and
        sqrt_three_continued_fraction_coefficients(Nat.3) = Nat.1 and
        sqrt_three_continued_fraction_coefficients(Nat.4) = Nat.2
} by {
    continued_fraction_periodic_sqrt_three_coefficient_zero
    sqrt_three_continued_fraction_coefficients(Nat.0) = Nat.1
    continued_fraction_periodic_sqrt_three_coefficient_one
    sqrt_three_continued_fraction_coefficients(Nat.1) = Nat.1
    continued_fraction_periodic_sqrt_three_coefficient_two
    sqrt_three_continued_fraction_coefficients(Nat.2) = Nat.2
    continued_fraction_periodic_sqrt_three_coefficient_three
    sqrt_three_continued_fraction_coefficients(Nat.3) = Nat.1
    continued_fraction_periodic_sqrt_three_coefficient_four
    sqrt_three_continued_fraction_coefficients(Nat.4) = Nat.2
    sqrt_three_continued_fraction_coefficients(Nat.0) = Nat.1 and
        sqrt_three_continued_fraction_coefficients(Nat.1) = Nat.1 and
        sqrt_three_continued_fraction_coefficients(Nat.2) = Nat.2 and
        sqrt_three_continued_fraction_coefficients(Nat.3) = Nat.1 and
        sqrt_three_continued_fraction_coefficients(Nat.4) = Nat.2
}

/// The odd partial quotients of √3 (after the integral part) are one:
/// a_{2k+1} = 1.
theorem continued_fraction_periodic_sqrt_three_coefficient_two_k_plus_one(k: Nat) {
    sqrt_three_continued_fraction_coefficients((Nat.2 * k).suc) = Nat.1
} by {
    Nat.0 < Nat.2
    mod_of_decomp(k, Nat.0, Nat.2)
    (k * Nat.2 + Nat.0).mod(Nat.2) = Nat.0
    add_zero_right(k * Nat.2)
    k * Nat.2 + Nat.0 = k * Nat.2
    mul_comm(k, Nat.2)
    k * Nat.2 = Nat.2 * k
    (Nat.2 * k).mod(Nat.2) = Nat.0
    sqrt_three_continued_fraction_coefficients((Nat.2 * k).suc) = Nat.1
}

/// The even partial quotients of √3 (from the second on) are two:
/// a_{2k+2} = 2.
theorem continued_fraction_periodic_sqrt_three_coefficient_two_k_plus_two(k: Nat) {
    sqrt_three_continued_fraction_coefficients((Nat.2 * k).suc.suc) = Nat.2
} by {
    Nat.1 < Nat.2
    mod_of_decomp(k, Nat.1, Nat.2)
    (k * Nat.2 + Nat.1).mod(Nat.2) = Nat.1
    mul_comm(k, Nat.2)
    k * Nat.2 = Nat.2 * k
    add_one_right(Nat.2 * k)
    Nat.2 * k + Nat.1 = (Nat.2 * k).suc
    ((Nat.2 * k).suc).mod(Nat.2) = Nat.1
    ((Nat.2 * k).suc).mod(Nat.2) != Nat.0
    sqrt_three_continued_fraction_coefficients((Nat.2 * k).suc.suc) = Nat.2
}

// The statement that every natural number is twice an index or one more than
// twice an index — needed to show that the coefficient sequence of √3 has a
// positive tail — is left for the end of this section.

/// Every natural number is twice an index or one more than twice an index.
theorem continued_fraction_periodic_nat_two_k_or_two_k_plus_one(n: Nat) {
    exists(k: Nat) {
        n = Nat.2 * k or n = Nat.2 * k + Nat.1
    }
} by {
    define p(m: Nat) -> Bool {
        exists(k: Nat) {
            m = Nat.2 * k or m = Nat.2 * k + Nat.1
        }
    }
    Nat.2 * Nat.0 = Nat.0
    Nat.0 = Nat.2 * Nat.0
    p(Nat.0)
    forall(m: Nat) {
        if p(m) {
            let k: Nat satisfy {
                m = Nat.2 * k or m = Nat.2 * k + Nat.1
            }
            if m = Nat.2 * k {
                add_one_right(Nat.2 * k)
                Nat.2 * k + Nat.1 = (Nat.2 * k).suc
                m.suc = (Nat.2 * k).suc
                m.suc = Nat.2 * k + Nat.1
                p(m.suc)
            }
            if not (m = Nat.2 * k) {
                m = Nat.2 * k or m = Nat.2 * k + Nat.1
                m = Nat.2 * k + Nat.1
                add_assoc(Nat.2 * k, Nat.1, Nat.1)
                (Nat.2 * k + Nat.1) + Nat.1 = Nat.2 * k + (Nat.1 + Nat.1)
                Nat.1 + Nat.1 = Nat.2
                (Nat.2 * k + Nat.1) + Nat.1 = Nat.2 * k + Nat.2
                add_one_right(Nat.2 * k + Nat.1)
                (Nat.2 * k + Nat.1) + Nat.1 = (Nat.2 * k + Nat.1).suc
                m.suc = (Nat.2 * k + Nat.1).suc
                m.suc = Nat.2 * k + Nat.2
                mul_suc_right(Nat.2, k)
                Nat.2 * k.suc = Nat.2 + Nat.2 * k
                add_comm(Nat.2, Nat.2 * k)
                Nat.2 + Nat.2 * k = Nat.2 * k + Nat.2
                Nat.2 * k + Nat.2 = Nat.2 * k.suc
                m.suc = Nat.2 * k.suc
                p(m.suc)
            }
            p(m.suc)
        }
        p(m) implies p(m.suc)
    }
    p(Nat.0) and forall(m: Nat) { p(m) implies p(m.suc) }
    Nat.induction(p)
    forall(m: Nat) { p(m) }
    p(n)
}

/// Every coefficient of the continued fraction of √3 after the integral part
/// is positive.
theorem continued_fraction_periodic_sqrt_three_coefficient_positive_suc(n: Nat) {
    Nat.0 < sqrt_three_continued_fraction_coefficients(n.suc)
} by {
    continued_fraction_periodic_nat_two_k_or_two_k_plus_one(n)
    let k: Nat satisfy {
        n = Nat.2 * k or n = Nat.2 * k + Nat.1
    }
    if n = Nat.2 * k {
        add_one_right(Nat.2 * k)
        Nat.2 * k + Nat.1 = (Nat.2 * k).suc
        n.suc = (Nat.2 * k).suc
        n.suc = Nat.2 * k + Nat.1
        continued_fraction_periodic_sqrt_three_coefficient_two_k_plus_one(k)
        sqrt_three_continued_fraction_coefficients((Nat.2 * k).suc) = Nat.1
        Nat.0 < Nat.1
        Nat.0 < sqrt_three_continued_fraction_coefficients(n.suc)
    }
    if not (n = Nat.2 * k) {
        n = Nat.2 * k or n = Nat.2 * k + Nat.1
        n = Nat.2 * k + Nat.1
        add_one_right(Nat.2 * k)
        Nat.2 * k + Nat.1 = (Nat.2 * k).suc
        continued_fraction_periodic_sqrt_three_coefficient_two_k_plus_two(k)
        sqrt_three_continued_fraction_coefficients((Nat.2 * k).suc.suc) = Nat.2
        n.suc = (Nat.2 * k + Nat.1).suc
        (Nat.2 * k + Nat.1).suc = (Nat.2 * k).suc.suc
        sqrt_three_continued_fraction_coefficients(n.suc) = Nat.2
        Nat.0 < Nat.2
        Nat.0 < sqrt_three_continued_fraction_coefficients(n.suc)
    }
    Nat.0 < sqrt_three_continued_fraction_coefficients(n.suc)
}

/// Every coefficient of the continued fraction of √3 after the integral part
/// is positive, so the coefficient sequence has a positive tail.
theorem continued_fraction_periodic_sqrt_three_positive_tail {
    positive_continued_fraction_sequence_tail(sqrt_three_continued_fraction_coefficients)
} by {
    forall(n: Nat) {
        continued_fraction_periodic_sqrt_three_coefficient_positive_suc(n)
        Nat.0 < sqrt_three_continued_fraction_coefficients(n.suc)
    }
    positive_continued_fraction_sequence_tail(sqrt_three_continued_fraction_coefficients)
}

// ============================================================================
// Section 3: the first convergents of √3
// ============================================================================

/// The zeroth convergent of √3 is 1/1.
theorem continued_fraction_periodic_sqrt_three_convergent_numerator_zero {
    continued_fraction_convergent_numerator(
        sqrt_three_continued_fraction_coefficients, Nat.0) = Nat.1
} by {
    continued_fraction_convergent_numerator_zero(
        sqrt_three_continued_fraction_coefficients)
    continued_fraction_convergent_numerator(
        sqrt_three_continued_fraction_coefficients, Nat.0) =
        sqrt_three_continued_fraction_coefficients(Nat.0)
    continued_fraction_periodic_sqrt_three_coefficient_zero
    sqrt_three_continued_fraction_coefficients(Nat.0) = Nat.1
    continued_fraction_convergent_numerator(
        sqrt_three_continued_fraction_coefficients, Nat.0) = Nat.1
}

/// The zeroth convergent of √3 has denominator one.
theorem continued_fraction_periodic_sqrt_three_convergent_denominator_zero {
    continued_fraction_convergent_denominator(
        sqrt_three_continued_fraction_coefficients, Nat.0) = Nat.1
} by {
    continued_fraction_convergent_denominator_zero(
        sqrt_three_continued_fraction_coefficients)
    continued_fraction_convergent_denominator(
        sqrt_three_continued_fraction_coefficients, Nat.0) = Nat.1
}

/// The first convergent of √3 has numerator two.
theorem continued_fraction_periodic_sqrt_three_convergent_numerator_one {
    continued_fraction_convergent_numerator(
        sqrt_three_continued_fraction_coefficients, Nat.1) = Nat.2
} by {
    continued_fraction_convergent_numerator_suc(
        sqrt_three_continued_fraction_coefficients, Nat.0)
    continued_fraction_convergent_numerator(
            sqrt_three_continued_fraction_coefficients, Nat.1) =
        continued_fraction_convergent_numerator(
                sqrt_three_continued_fraction_coefficients, Nat.0) *
            sqrt_three_continued_fraction_coefficients(Nat.1) +
        continued_fraction_recurrence_state(
            sqrt_three_continued_fraction_coefficients, Nat.1, Nat.0, Nat.1).first
    continued_fraction_convergent_numerator_zero(
        sqrt_three_continued_fraction_coefficients)
    continued_fraction_convergent_numerator(
        sqrt_three_continued_fraction_coefficients, Nat.0) =
        sqrt_three_continued_fraction_coefficients(Nat.0)
    continued_fraction_periodic_sqrt_three_coefficient_zero
    sqrt_three_continued_fraction_coefficients(Nat.0) = Nat.1
    continued_fraction_convergent_numerator(
        sqrt_three_continued_fraction_coefficients, Nat.0) = Nat.1
    continued_fraction_recurrence_state_suc_first(
        sqrt_three_continued_fraction_coefficients, Nat.0, Nat.0, Nat.1)
    continued_fraction_recurrence_state(
            sqrt_three_continued_fraction_coefficients, Nat.1, Nat.0, Nat.1).first =
        continued_fraction_recurrence_state(
            sqrt_three_continued_fraction_coefficients, Nat.0, Nat.0, Nat.1).second
    continued_fraction_recurrence_state_zero(
        sqrt_three_continued_fraction_coefficients, Nat.0, Nat.1)
    continued_fraction_recurrence_state(
        sqrt_three_continued_fraction_coefficients, Nat.0, Nat.0, Nat.1) =
        Pair.new(Nat.0, Nat.1)
    pair_new_second(Nat.0, Nat.1)
    continued_fraction_recurrence_state(
        sqrt_three_continued_fraction_coefficients, Nat.0, Nat.0, Nat.1).second =
        Nat.1
    continued_fraction_recurrence_state(
        sqrt_three_continued_fraction_coefficients, Nat.1, Nat.0, Nat.1).first =
        Nat.1
    continued_fraction_periodic_sqrt_three_coefficient_one
    sqrt_three_continued_fraction_coefficients(Nat.1) = Nat.1
    continued_fraction_convergent_numerator(
            sqrt_three_continued_fraction_coefficients, Nat.1) =
        Nat.1 * Nat.1 + Nat.1
    Nat.1 * Nat.1 + Nat.1 = Nat.2
    continued_fraction_convergent_numerator(
        sqrt_three_continued_fraction_coefficients, Nat.1) = Nat.2
}

/// The first convergent of √3 has denominator one.
theorem continued_fraction_periodic_sqrt_three_convergent_denominator_one {
    continued_fraction_convergent_denominator(
        sqrt_three_continued_fraction_coefficients, Nat.1) = Nat.1
} by {
    continued_fraction_convergent_denominator_suc(
        sqrt_three_continued_fraction_coefficients, Nat.0)
    continued_fraction_convergent_denominator(
            sqrt_three_continued_fraction_coefficients, Nat.1) =
        continued_fraction_convergent_denominator(
                sqrt_three_continued_fraction_coefficients, Nat.0) *
            sqrt_three_continued_fraction_coefficients(Nat.1) +
        continued_fraction_recurrence_state(
            sqrt_three_continued_fraction_coefficients, Nat.1, Nat.1, Nat.0).first
    continued_fraction_convergent_denominator_zero(
        sqrt_three_continued_fraction_coefficients)
    continued_fraction_convergent_denominator(
        sqrt_three_continued_fraction_coefficients, Nat.0) = Nat.1
    continued_fraction_recurrence_state_suc_first(
        sqrt_three_continued_fraction_coefficients, Nat.0, Nat.1, Nat.0)
    continued_fraction_recurrence_state(
            sqrt_three_continued_fraction_coefficients, Nat.1, Nat.1, Nat.0).first =
        continued_fraction_recurrence_state(
            sqrt_three_continued_fraction_coefficients, Nat.0, Nat.1, Nat.0).second
    continued_fraction_recurrence_state_zero(
        sqrt_three_continued_fraction_coefficients, Nat.1, Nat.0)
    continued_fraction_recurrence_state(
        sqrt_three_continued_fraction_coefficients, Nat.0, Nat.1, Nat.0) =
        Pair.new(Nat.1, Nat.0)
    pair_new_second(Nat.1, Nat.0)
    continued_fraction_recurrence_state(
        sqrt_three_continued_fraction_coefficients, Nat.0, Nat.1, Nat.0).second =
        Nat.0
    continued_fraction_recurrence_state(
        sqrt_three_continued_fraction_coefficients, Nat.1, Nat.1, Nat.0).first =
        Nat.0
    continued_fraction_periodic_sqrt_three_coefficient_one
    sqrt_three_continued_fraction_coefficients(Nat.1) = Nat.1
    continued_fraction_convergent_denominator(
            sqrt_three_continued_fraction_coefficients, Nat.1) =
        Nat.1 * Nat.1 + Nat.0
    Nat.1 * Nat.1 + Nat.0 = Nat.1
    continued_fraction_convergent_denominator(
        sqrt_three_continued_fraction_coefficients, Nat.1) = Nat.1
}

/// The second convergent of √3 has numerator five.
theorem continued_fraction_periodic_sqrt_three_convergent_numerator_two {
    continued_fraction_convergent_numerator(
        sqrt_three_continued_fraction_coefficients, Nat.2) = Nat.5
} by {
    continued_fraction_convergent_numerator_two_suc(
        sqrt_three_continued_fraction_coefficients, Nat.0)
    continued_fraction_convergent_numerator(
            sqrt_three_continued_fraction_coefficients, Nat.0.suc.suc) =
        continued_fraction_convergent_numerator(
                sqrt_three_continued_fraction_coefficients, Nat.0.suc) *
            sqrt_three_continued_fraction_coefficients(Nat.0.suc.suc) +
        continued_fraction_convergent_numerator(
            sqrt_three_continued_fraction_coefficients, Nat.0)
    Nat.0.suc = Nat.1
    Nat.0.suc.suc = Nat.2
    continued_fraction_convergent_numerator(
            sqrt_three_continued_fraction_coefficients, Nat.2) =
        continued_fraction_convergent_numerator(
                sqrt_three_continued_fraction_coefficients, Nat.1) *
            sqrt_three_continued_fraction_coefficients(Nat.2) +
        continued_fraction_convergent_numerator(
            sqrt_three_continued_fraction_coefficients, Nat.0)
    continued_fraction_periodic_sqrt_three_convergent_numerator_one
    continued_fraction_convergent_numerator(
        sqrt_three_continued_fraction_coefficients, Nat.1) = Nat.2
    continued_fraction_periodic_sqrt_three_coefficient_two
    sqrt_three_continued_fraction_coefficients(Nat.2) = Nat.2
    continued_fraction_periodic_sqrt_three_convergent_numerator_zero
    continued_fraction_convergent_numerator(
        sqrt_three_continued_fraction_coefficients, Nat.0) = Nat.1
    continued_fraction_convergent_numerator(
            sqrt_three_continued_fraction_coefficients, Nat.2) =
        Nat.2 * Nat.2 + Nat.1
    Nat.2 * Nat.2 + Nat.1 = Nat.5
    continued_fraction_convergent_numerator(
        sqrt_three_continued_fraction_coefficients, Nat.2) = Nat.5
}

/// The second convergent of √3 has denominator three.
theorem continued_fraction_periodic_sqrt_three_convergent_denominator_two {
    continued_fraction_convergent_denominator(
        sqrt_three_continued_fraction_coefficients, Nat.2) = Nat.3
} by {
    continued_fraction_convergent_denominator_two_suc(
        sqrt_three_continued_fraction_coefficients, Nat.0)
    continued_fraction_convergent_denominator(
            sqrt_three_continued_fraction_coefficients, Nat.0.suc.suc) =
        continued_fraction_convergent_denominator(
                sqrt_three_continued_fraction_coefficients, Nat.0.suc) *
            sqrt_three_continued_fraction_coefficients(Nat.0.suc.suc) +
        continued_fraction_convergent_denominator(
            sqrt_three_continued_fraction_coefficients, Nat.0)
    Nat.0.suc = Nat.1
    Nat.0.suc.suc = Nat.2
    continued_fraction_convergent_denominator(
            sqrt_three_continued_fraction_coefficients, Nat.2) =
        continued_fraction_convergent_denominator(
                sqrt_three_continued_fraction_coefficients, Nat.1) *
            sqrt_three_continued_fraction_coefficients(Nat.2) +
        continued_fraction_convergent_denominator(
            sqrt_three_continued_fraction_coefficients, Nat.0)
    continued_fraction_periodic_sqrt_three_convergent_denominator_one
    continued_fraction_convergent_denominator(
        sqrt_three_continued_fraction_coefficients, Nat.1) = Nat.1
    continued_fraction_periodic_sqrt_three_coefficient_two
    sqrt_three_continued_fraction_coefficients(Nat.2) = Nat.2
    continued_fraction_periodic_sqrt_three_convergent_denominator_zero
    continued_fraction_convergent_denominator(
        sqrt_three_continued_fraction_coefficients, Nat.0) = Nat.1
    continued_fraction_convergent_denominator(
            sqrt_three_continued_fraction_coefficients, Nat.2) =
        Nat.1 * Nat.2 + Nat.1
    Nat.1 * Nat.2 + Nat.1 = Nat.3
    continued_fraction_convergent_denominator(
        sqrt_three_continued_fraction_coefficients, Nat.2) = Nat.3
}

/// The third convergent of √3 has numerator seven.
theorem continued_fraction_periodic_sqrt_three_convergent_numerator_three {
    continued_fraction_convergent_numerator(
        sqrt_three_continued_fraction_coefficients, Nat.3) = Nat.7
} by {
    continued_fraction_convergent_numerator_two_suc(
        sqrt_three_continued_fraction_coefficients, Nat.1)
    continued_fraction_convergent_numerator(
            sqrt_three_continued_fraction_coefficients, Nat.1.suc.suc) =
        continued_fraction_convergent_numerator(
                sqrt_three_continued_fraction_coefficients, Nat.1.suc) *
            sqrt_three_continued_fraction_coefficients(Nat.1.suc.suc) +
        continued_fraction_convergent_numerator(
            sqrt_three_continued_fraction_coefficients, Nat.1)
    Nat.1.suc = Nat.2
    Nat.1.suc.suc = Nat.3
    continued_fraction_convergent_numerator(
            sqrt_three_continued_fraction_coefficients, Nat.3) =
        continued_fraction_convergent_numerator(
                sqrt_three_continued_fraction_coefficients, Nat.2) *
            sqrt_three_continued_fraction_coefficients(Nat.3) +
        continued_fraction_convergent_numerator(
            sqrt_three_continued_fraction_coefficients, Nat.1)
    continued_fraction_periodic_sqrt_three_convergent_numerator_two
    continued_fraction_convergent_numerator(
        sqrt_three_continued_fraction_coefficients, Nat.2) = Nat.5
    continued_fraction_periodic_sqrt_three_coefficient_three
    sqrt_three_continued_fraction_coefficients(Nat.3) = Nat.1
    continued_fraction_periodic_sqrt_three_convergent_numerator_one
    continued_fraction_convergent_numerator(
        sqrt_three_continued_fraction_coefficients, Nat.1) = Nat.2
    continued_fraction_convergent_numerator(
            sqrt_three_continued_fraction_coefficients, Nat.3) =
        Nat.5 * Nat.1 + Nat.2
    Nat.5 * Nat.1 = Nat.5
    Nat.5 + Nat.2 = Nat.7
    Nat.5 * Nat.1 + Nat.2 = Nat.7
    continued_fraction_convergent_numerator(
        sqrt_three_continued_fraction_coefficients, Nat.3) = Nat.7
}

/// The third convergent of √3 has denominator four.
theorem continued_fraction_periodic_sqrt_three_convergent_denominator_three {
    continued_fraction_convergent_denominator(
        sqrt_three_continued_fraction_coefficients, Nat.3) = Nat.4
} by {
    continued_fraction_convergent_denominator_two_suc(
        sqrt_three_continued_fraction_coefficients, Nat.1)
    continued_fraction_convergent_denominator(
            sqrt_three_continued_fraction_coefficients, Nat.1.suc.suc) =
        continued_fraction_convergent_denominator(
                sqrt_three_continued_fraction_coefficients, Nat.1.suc) *
            sqrt_three_continued_fraction_coefficients(Nat.1.suc.suc) +
        continued_fraction_convergent_denominator(
            sqrt_three_continued_fraction_coefficients, Nat.1)
    Nat.1.suc = Nat.2
    Nat.1.suc.suc = Nat.3
    continued_fraction_convergent_denominator(
            sqrt_three_continued_fraction_coefficients, Nat.3) =
        continued_fraction_convergent_denominator(
                sqrt_three_continued_fraction_coefficients, Nat.2) *
            sqrt_three_continued_fraction_coefficients(Nat.3) +
        continued_fraction_convergent_denominator(
            sqrt_three_continued_fraction_coefficients, Nat.1)
    continued_fraction_periodic_sqrt_three_convergent_denominator_two
    continued_fraction_convergent_denominator(
        sqrt_three_continued_fraction_coefficients, Nat.2) = Nat.3
    continued_fraction_periodic_sqrt_three_coefficient_three
    sqrt_three_continued_fraction_coefficients(Nat.3) = Nat.1
    continued_fraction_periodic_sqrt_three_convergent_denominator_one
    continued_fraction_convergent_denominator(
        sqrt_three_continued_fraction_coefficients, Nat.1) = Nat.1
    continued_fraction_convergent_denominator(
            sqrt_three_continued_fraction_coefficients, Nat.3) =
        Nat.3 * Nat.1 + Nat.1
    Nat.3 * Nat.1 + Nat.1 = Nat.4
    continued_fraction_convergent_denominator(
        sqrt_three_continued_fraction_coefficients, Nat.3) = Nat.4
}

/// The first four convergents of √3 are 1/1, 2/1, 5/3, 7/4.
theorem continued_fraction_periodic_sqrt_three_first_convergents {
    continued_fraction_convergent_numerator(
            sqrt_three_continued_fraction_coefficients, Nat.0) = Nat.1 and
    continued_fraction_convergent_denominator(
            sqrt_three_continued_fraction_coefficients, Nat.0) = Nat.1 and
    continued_fraction_convergent_numerator(
            sqrt_three_continued_fraction_coefficients, Nat.1) = Nat.2 and
    continued_fraction_convergent_denominator(
            sqrt_three_continued_fraction_coefficients, Nat.1) = Nat.1 and
    continued_fraction_convergent_numerator(
            sqrt_three_continued_fraction_coefficients, Nat.2) = Nat.5 and
    continued_fraction_convergent_denominator(
            sqrt_three_continued_fraction_coefficients, Nat.2) = Nat.3 and
    continued_fraction_convergent_numerator(
            sqrt_three_continued_fraction_coefficients, Nat.3) = Nat.7 and
    continued_fraction_convergent_denominator(
            sqrt_three_continued_fraction_coefficients, Nat.3) = Nat.4
} by {
    continued_fraction_periodic_sqrt_three_convergent_numerator_zero
    continued_fraction_convergent_numerator(
        sqrt_three_continued_fraction_coefficients, Nat.0) = Nat.1
    continued_fraction_periodic_sqrt_three_convergent_denominator_zero
    continued_fraction_convergent_denominator(
        sqrt_three_continued_fraction_coefficients, Nat.0) = Nat.1
    continued_fraction_periodic_sqrt_three_convergent_numerator_one
    continued_fraction_convergent_numerator(
        sqrt_three_continued_fraction_coefficients, Nat.1) = Nat.2
    continued_fraction_periodic_sqrt_three_convergent_denominator_one
    continued_fraction_convergent_denominator(
        sqrt_three_continued_fraction_coefficients, Nat.1) = Nat.1
    continued_fraction_periodic_sqrt_three_convergent_numerator_two
    continued_fraction_convergent_numerator(
        sqrt_three_continued_fraction_coefficients, Nat.2) = Nat.5
    continued_fraction_periodic_sqrt_three_convergent_denominator_two
    continued_fraction_convergent_denominator(
        sqrt_three_continued_fraction_coefficients, Nat.2) = Nat.3
    continued_fraction_periodic_sqrt_three_convergent_numerator_three
    continued_fraction_convergent_numerator(
        sqrt_three_continued_fraction_coefficients, Nat.3) = Nat.7
    continued_fraction_periodic_sqrt_three_convergent_denominator_three
    continued_fraction_convergent_denominator(
        sqrt_three_continued_fraction_coefficients, Nat.3) = Nat.4
    continued_fraction_convergent_numerator(
            sqrt_three_continued_fraction_coefficients, Nat.0) = Nat.1 and
    continued_fraction_convergent_denominator(
            sqrt_three_continued_fraction_coefficients, Nat.0) = Nat.1 and
    continued_fraction_convergent_numerator(
            sqrt_three_continued_fraction_coefficients, Nat.1) = Nat.2 and
    continued_fraction_convergent_denominator(
            sqrt_three_continued_fraction_coefficients, Nat.1) = Nat.1 and
    continued_fraction_convergent_numerator(
            sqrt_three_continued_fraction_coefficients, Nat.2) = Nat.5 and
    continued_fraction_convergent_denominator(
            sqrt_three_continued_fraction_coefficients, Nat.2) = Nat.3 and
    continued_fraction_convergent_numerator(
            sqrt_three_continued_fraction_coefficients, Nat.3) = Nat.7 and
    continued_fraction_convergent_denominator(
            sqrt_three_continued_fraction_coefficients, Nat.3) = Nat.4
}

/// The first convergent of √3 is the rational 2/1.
theorem continued_fraction_periodic_sqrt_three_convergent_value_one {
    continued_fraction_convergent_value(
        sqrt_three_continued_fraction_coefficients, Nat.1) =
        Rat.from_nat(Nat.2) / Rat.from_nat(Nat.1)
} by {
    continued_fraction_convergent_value(
            sqrt_three_continued_fraction_coefficients, Nat.1) =
        Rat.from_nat(continued_fraction_convergent_numerator(
            sqrt_three_continued_fraction_coefficients, Nat.1)) /
        Rat.from_nat(continued_fraction_convergent_denominator(
            sqrt_three_continued_fraction_coefficients, Nat.1))
    continued_fraction_periodic_sqrt_three_convergent_numerator_one
    continued_fraction_convergent_numerator(
        sqrt_three_continued_fraction_coefficients, Nat.1) = Nat.2
    continued_fraction_periodic_sqrt_three_convergent_denominator_one
    continued_fraction_convergent_denominator(
        sqrt_three_continued_fraction_coefficients, Nat.1) = Nat.1
    continued_fraction_convergent_value(
        sqrt_three_continued_fraction_coefficients, Nat.1) =
        Rat.from_nat(Nat.2) / Rat.from_nat(Nat.1)
}

/// The second convergent of √3 is the rational 5/3.
theorem continued_fraction_periodic_sqrt_three_convergent_value_two {
    continued_fraction_convergent_value(
        sqrt_three_continued_fraction_coefficients, Nat.2) =
        Rat.from_nat(Nat.5) / Rat.from_nat(Nat.3)
} by {
    continued_fraction_convergent_value(
            sqrt_three_continued_fraction_coefficients, Nat.2) =
        Rat.from_nat(continued_fraction_convergent_numerator(
            sqrt_three_continued_fraction_coefficients, Nat.2)) /
        Rat.from_nat(continued_fraction_convergent_denominator(
            sqrt_three_continued_fraction_coefficients, Nat.2))
    continued_fraction_periodic_sqrt_three_convergent_numerator_two
    continued_fraction_convergent_numerator(
        sqrt_three_continued_fraction_coefficients, Nat.2) = Nat.5
    continued_fraction_periodic_sqrt_three_convergent_denominator_two
    continued_fraction_convergent_denominator(
        sqrt_three_continued_fraction_coefficients, Nat.2) = Nat.3
    continued_fraction_convergent_value(
        sqrt_three_continued_fraction_coefficients, Nat.2) =
        Rat.from_nat(Nat.5) / Rat.from_nat(Nat.3)
}

/// The third convergent of √3 is the rational 7/4.
theorem continued_fraction_periodic_sqrt_three_convergent_value_three {
    continued_fraction_convergent_value(
        sqrt_three_continued_fraction_coefficients, Nat.3) =
        Rat.from_nat(Nat.7) / Rat.from_nat(Nat.4)
} by {
    continued_fraction_convergent_value(
            sqrt_three_continued_fraction_coefficients, Nat.3) =
        Rat.from_nat(continued_fraction_convergent_numerator(
            sqrt_three_continued_fraction_coefficients, Nat.3)) /
        Rat.from_nat(continued_fraction_convergent_denominator(
            sqrt_three_continued_fraction_coefficients, Nat.3))
    continued_fraction_periodic_sqrt_three_convergent_numerator_three
    continued_fraction_convergent_numerator(
        sqrt_three_continued_fraction_coefficients, Nat.3) = Nat.7
    continued_fraction_periodic_sqrt_three_convergent_denominator_three
    continued_fraction_convergent_denominator(
        sqrt_three_continued_fraction_coefficients, Nat.3) = Nat.4
    continued_fraction_convergent_value(
        sqrt_three_continued_fraction_coefficients, Nat.3) =
        Rat.from_nat(Nat.7) / Rat.from_nat(Nat.4)
}

// ============================================================================
// Section 4: the norms p_n² - D·q_n² of the convergents
// ============================================================================

/// The norm of the n-th convergent of √2 is the alternating sign at the next
/// index: p_n² - 2·q_n² = (-1)^(n+1).
theorem continued_fraction_periodic_sqrt_two_norm_alternating(n: Nat) {
    pell_norm(Int.2,
        Int.from_nat(continued_fraction_convergent_numerator(
            sqrt_two_continued_fraction_coefficients, n)),
        Int.from_nat(continued_fraction_convergent_denominator(
            sqrt_two_continued_fraction_coefficients, n))) =
        alternating_sign[Int](n.suc)
} by {
    cf_pell_convergent_norm_alternating(n)
    pell_norm(Int.2,
        Int.from_nat(continued_fraction_convergent_numerator(
            sqrt_two_continued_fraction_coefficients, n)),
        Int.from_nat(continued_fraction_convergent_denominator(
            sqrt_two_continued_fraction_coefficients, n))) =
        alternating_sign[Int](n.suc)
}

/// Every convergent of √2 has Pell norm one or minus one:
/// p_n² - 2·q_n² = ±1.
theorem continued_fraction_periodic_sqrt_two_norm_one_or_neg_one(n: Nat) {
    pell_norm(Int.2,
        Int.from_nat(continued_fraction_convergent_numerator(
            sqrt_two_continued_fraction_coefficients, n)),
        Int.from_nat(continued_fraction_convergent_denominator(
            sqrt_two_continued_fraction_coefficients, n))) = Int.1 or
    pell_norm(Int.2,
        Int.from_nat(continued_fraction_convergent_numerator(
            sqrt_two_continued_fraction_coefficients, n)),
        Int.from_nat(continued_fraction_convergent_denominator(
            sqrt_two_continued_fraction_coefficients, n))) = -Int.1
} by {
    cf_pell_convergent_norm_one_or_neg_one(n)
    pell_norm(Int.2,
        Int.from_nat(continued_fraction_convergent_numerator(
            sqrt_two_continued_fraction_coefficients, n)),
        Int.from_nat(continued_fraction_convergent_denominator(
            sqrt_two_continued_fraction_coefficients, n))) = Int.1 or
    pell_norm(Int.2,
        Int.from_nat(continued_fraction_convergent_numerator(
            sqrt_two_continued_fraction_coefficients, n)),
        Int.from_nat(continued_fraction_convergent_denominator(
            sqrt_two_continued_fraction_coefficients, n))) = -Int.1
}

/// The absolute value of the Pell norm of a convergent of √2 is one:
/// |p_n² - 2·q_n²| = 1.
theorem continued_fraction_periodic_sqrt_two_norm_abs_one(n: Nat) {
    abs(pell_norm(Int.2,
        Int.from_nat(continued_fraction_convergent_numerator(
            sqrt_two_continued_fraction_coefficients, n)),
        Int.from_nat(continued_fraction_convergent_denominator(
            sqrt_two_continued_fraction_coefficients, n)))) = Nat.1
} by {
    cf_pell_convergent_norm_abs_one(n)
    abs(pell_norm(Int.2,
        Int.from_nat(continued_fraction_convergent_numerator(
            sqrt_two_continued_fraction_coefficients, n)),
        Int.from_nat(continued_fraction_convergent_denominator(
            sqrt_two_continued_fraction_coefficients, n)))) = Nat.1
}

/// The norms of the convergents of √2 alternate with period two:
/// p_{n+2}² - 2·q_{n+2}² = p_n² - 2·q_n².
theorem continued_fraction_periodic_sqrt_two_norm_period_two(n: Nat) {
    pell_norm(Int.2,
        Int.from_nat(continued_fraction_convergent_numerator(
            sqrt_two_continued_fraction_coefficients, n.suc.suc)),
        Int.from_nat(continued_fraction_convergent_denominator(
            sqrt_two_continued_fraction_coefficients, n.suc.suc))) =
    pell_norm(Int.2,
        Int.from_nat(continued_fraction_convergent_numerator(
            sqrt_two_continued_fraction_coefficients, n)),
        Int.from_nat(continued_fraction_convergent_denominator(
            sqrt_two_continued_fraction_coefficients, n)))
} by {
    cf_pell_convergent_norm_alternating(n.suc.suc)
    pell_norm(Int.2,
        Int.from_nat(continued_fraction_convergent_numerator(
            sqrt_two_continued_fraction_coefficients, n.suc.suc)),
        Int.from_nat(continued_fraction_convergent_denominator(
            sqrt_two_continued_fraction_coefficients, n.suc.suc))) =
        alternating_sign[Int](n.suc.suc.suc)
    cf_pell_convergent_norm_alternating(n)
    pell_norm(Int.2,
        Int.from_nat(continued_fraction_convergent_numerator(
            sqrt_two_continued_fraction_coefficients, n)),
        Int.from_nat(continued_fraction_convergent_denominator(
            sqrt_two_continued_fraction_coefficients, n))) =
        alternating_sign[Int](n.suc)
    alternating_sign_suc[Int](n.suc)
    alternating_sign[Int](n.suc.suc) = -alternating_sign[Int](n.suc)
    alternating_sign_suc[Int](n.suc.suc)
    alternating_sign[Int](n.suc.suc.suc) = -alternating_sign[Int](n.suc.suc)
    alternating_sign[Int](n.suc.suc.suc) = --alternating_sign[Int](n.suc)
    --alternating_sign[Int](n.suc) = alternating_sign[Int](n.suc)
    alternating_sign[Int](n.suc.suc.suc) = alternating_sign[Int](n.suc)
    pell_norm(Int.2,
        Int.from_nat(continued_fraction_convergent_numerator(
            sqrt_two_continued_fraction_coefficients, n.suc.suc)),
        Int.from_nat(continued_fraction_convergent_denominator(
            sqrt_two_continued_fraction_coefficients, n.suc.suc))) =
        alternating_sign[Int](n.suc)
    pell_norm(Int.2,
        Int.from_nat(continued_fraction_convergent_numerator(
            sqrt_two_continued_fraction_coefficients, n.suc.suc)),
        Int.from_nat(continued_fraction_convergent_denominator(
            sqrt_two_continued_fraction_coefficients, n.suc.suc))) =
    pell_norm(Int.2,
        Int.from_nat(continued_fraction_convergent_numerator(
            sqrt_two_continued_fraction_coefficients, n)),
        Int.from_nat(continued_fraction_convergent_denominator(
            sqrt_two_continued_fraction_coefficients, n)))
}

/// (a + b) - b = a.
theorem continued_fraction_periodic_sub_add_cancel(a: Int, b: Int) {
    (a + b) - b = a
}

/// 1 - 3 = -2.
theorem continued_fraction_periodic_int_one_sub_three {
    Int.1 - Int.3 = -Int.2
} by {
    Int.1 - Int.3 = sub_nat(Nat.1, Nat.3)
    neg_sub_nat(Nat.1, Nat.3)
    sub_nat(Nat.1, Nat.3) = -(sub_nat(Nat.3, Nat.1))
    Nat.2 + Nat.1 = Nat.3
    sub_nat_add_left(Nat.2, Nat.1)
    sub_nat(Nat.2 + Nat.1, Nat.1) = Int.from_nat(Nat.2)
    sub_nat(Nat.3, Nat.1) = Int.from_nat(Nat.2)
    sub_nat(Nat.1, Nat.3) = -(Int.from_nat(Nat.2))
    Int.from_nat(Nat.2) = Int.2
    sub_nat(Nat.1, Nat.3) = -Int.2
    Int.1 - Int.3 = -Int.2
}

/// 4 - 3 = 1.
theorem continued_fraction_periodic_int_four_sub_three {
    Int.4 - Int.3 = Int.1
} by {
    Int.4 - Int.3 = sub_nat(Nat.4, Nat.3)
    Nat.1 + Nat.3 = Nat.4
    sub_nat_add_left(Nat.1, Nat.3)
    sub_nat(Nat.1 + Nat.3, Nat.3) = Int.from_nat(Nat.1)
    sub_nat(Nat.4, Nat.3) = Int.from_nat(Nat.1)
    Int.from_nat(Nat.1) = Int.1
    sub_nat(Nat.4, Nat.3) = Int.1
    Int.4 - Int.3 = Int.1
}

/// 3 - 4 = -1.
theorem continued_fraction_periodic_int_three_sub_four {
    Int.3 - Int.4 = -Int.1
} by {
    continued_fraction_periodic_int_four_sub_three
    Int.4 - Int.3 = Int.1
    neg_sub(Int.3, Int.4)
    Int.3 - Int.4 = -(Int.4 - Int.3)
    Int.3 - Int.4 = -Int.1
}

/// 3·2 = 6.
theorem continued_fraction_periodic_int_three_mul_two {
    Int.3 * Int.2 = Int.6
} by {
    mul_from_nat(Nat.3, Nat.2)
    Int.from_nat(Nat.3 * Nat.2) = Int.from_nat(Nat.3) * Int.from_nat(Nat.2)
    Nat.3 * Nat.2 = Nat.6
    Int.from_nat(Nat.6) = Int.from_nat(Nat.3) * Int.from_nat(Nat.2)
    Int.from_nat(Nat.3) = Int.3
    Int.from_nat(Nat.2) = Int.2
    Int.from_nat(Nat.6) = Int.6
    Int.6 = Int.3 * Int.2
    Int.3 * Int.2 = Int.6
}

/// 2·3 = 6.
theorem continued_fraction_periodic_int_two_mul_three {
    Int.2 * Int.3 = Int.6
} by {
    mul_from_nat(Nat.2, Nat.3)
    Int.from_nat(Nat.2 * Nat.3) = Int.from_nat(Nat.2) * Int.from_nat(Nat.3)
    Nat.2 * Nat.3 = Nat.6
    Int.from_nat(Nat.6) = Int.from_nat(Nat.2) * Int.from_nat(Nat.3)
    Int.from_nat(Nat.2) = Int.2
    Int.from_nat(Nat.3) = Int.3
    Int.from_nat(Nat.6) = Int.6
    Int.6 = Int.2 * Int.3
    Int.2 * Int.3 = Int.6
}

/// 3·3 = 9.
theorem continued_fraction_periodic_int_three_mul_three {
    Int.3 * Int.3 = Int.9
} by {
    mul_from_nat(Nat.3, Nat.3)
    Int.from_nat(Nat.3 * Nat.3) = Int.from_nat(Nat.3) * Int.from_nat(Nat.3)
    Nat.3 * Nat.3 = Nat.9
    Int.from_nat(Nat.9) = Int.from_nat(Nat.3) * Int.from_nat(Nat.3)
    Int.from_nat(Nat.3) = Int.3
    Int.from_nat(Nat.9) = Int.9
    Int.9 = Int.3 * Int.3
    Int.3 * Int.3 = Int.9
}

/// 2·2 = 4.
theorem continued_fraction_periodic_int_two_mul_two {
    Int.2 * Int.2 = Int.4
} by {
    mul_from_nat(Nat.2, Nat.2)
    Int.from_nat(Nat.2 * Nat.2) = Int.from_nat(Nat.2) * Int.from_nat(Nat.2)
    Nat.2 * Nat.2 = Nat.4
    Int.from_nat(Nat.4) = Int.from_nat(Nat.2) * Int.from_nat(Nat.2)
    Int.from_nat(Nat.2) = Int.2
    Int.from_nat(Nat.4) = Int.4
    Int.4 = Int.2 * Int.2
    Int.2 * Int.2 = Int.4
}

/// 2 + 2 = 4.
theorem continued_fraction_periodic_int_two_add_two {
    Int.2 + Int.2 = Int.4
} by {
    add_from_nat(Nat.2, Nat.2)
    Int.from_nat(Nat.2) + Int.from_nat(Nat.2) = Int.from_nat(Nat.2 + Nat.2)
    Nat.2 + Nat.2 = Nat.4
    Int.from_nat(Nat.2) + Int.from_nat(Nat.2) = Int.from_nat(Nat.4)
    Int.from_nat(Nat.2) = Int.2
    Int.from_nat(Nat.4) = Int.4
    Int.2 + Int.2 = Int.4
}

/// (2x)² = 4x².
theorem continued_fraction_periodic_int_two_x_sq(x: Int) {
    (Int.2 * x) * (Int.2 * x) = Int.4 * (x * x)
} by {
    (Int.2 * x) * (Int.2 * x) = Int.2 * (x * (Int.2 * x))
    x * (Int.2 * x) = Int.2 * (x * x)
    Int.2 * (Int.2 * (x * x)) = (Int.2 * Int.2) * (x * x)
    continued_fraction_periodic_int_two_mul_two
    Int.2 * Int.2 = Int.4
    (Int.2 * Int.2) * (x * x) = Int.4 * (x * x)
    Int.2 * (Int.2 * (x * x)) = Int.4 * (x * x)
    (Int.2 * x) * (Int.2 * x) = Int.4 * (x * x)
}

/// (3y)·(2x) = 6·(x·y).
theorem continued_fraction_periodic_int_three_y_two_x(x: Int, y: Int) {
    (Int.3 * y) * (Int.2 * x) = Int.6 * (x * y)
} by {
    (Int.3 * y) * (Int.2 * x) = Int.3 * (y * (Int.2 * x))
    y * (Int.2 * x) = (y * Int.2) * x
    mul_assoc(y, Int.2, x)
    y * Int.2 = Int.2 * y
    (Int.2 * y) * x = Int.2 * (y * x)
    mul_assoc(Int.2, y, x)
    y * (Int.2 * x) = Int.2 * (y * x)
    y * x = x * y
    Int.3 * (Int.2 * (x * y)) = (Int.3 * Int.2) * (x * y)
    continued_fraction_periodic_int_three_mul_two
    Int.3 * Int.2 = Int.6
    (Int.3 * Int.2) * (x * y) = Int.6 * (x * y)
    Int.3 * (Int.2 * (x * y)) = Int.6 * (x * y)
    (Int.3 * y) * (Int.2 * x) = Int.6 * (x * y)
}

/// (2x)·(3y) = 6·(x·y).
theorem continued_fraction_periodic_int_two_x_three_y(x: Int, y: Int) {
    (Int.2 * x) * (Int.3 * y) = Int.6 * (x * y)
} by {
    (Int.2 * x) * (Int.3 * y) = Int.2 * (x * (Int.3 * y))
    x * (Int.3 * y) = (x * Int.3) * y
    mul_assoc(x, Int.3, y)
    x * Int.3 = Int.3 * x
    (Int.3 * x) * y = Int.3 * (x * y)
    mul_assoc(Int.3, x, y)
    x * (Int.3 * y) = Int.3 * (x * y)
    Int.2 * (Int.3 * (x * y)) = (Int.2 * Int.3) * (x * y)
    continued_fraction_periodic_int_two_mul_three
    Int.2 * Int.3 = Int.6
    (Int.2 * Int.3) * (x * y) = Int.6 * (x * y)
    Int.2 * (Int.3 * (x * y)) = Int.6 * (x * y)
    (Int.2 * x) * (Int.3 * y) = Int.6 * (x * y)
}

/// (3y)² = 9y².
theorem continued_fraction_periodic_int_three_y_sq(y: Int) {
    (Int.3 * y) * (Int.3 * y) = Int.9 * (y * y)
} by {
    (Int.3 * y) * (Int.3 * y) = Int.3 * (y * (Int.3 * y))
    y * (Int.3 * y) = Int.3 * (y * y)
    Int.3 * (Int.3 * (y * y)) = (Int.3 * Int.3) * (y * y)
    continued_fraction_periodic_int_three_mul_three
    Int.3 * Int.3 = Int.9
    (Int.3 * Int.3) * (y * y) = Int.9 * (y * y)
    Int.3 * (Int.3 * (y * y)) = Int.9 * (y * y)
    (Int.3 * y) * (Int.3 * y) = Int.9 * (y * y)
}

/// (2y)² = 4y².
theorem continued_fraction_periodic_int_two_y_sq(y: Int) {
    (Int.2 * y) * (Int.2 * y) = Int.4 * (y * y)
} by {
    continued_fraction_periodic_int_two_x_sq(y)
    (Int.2 * y) * (Int.2 * y) = Int.4 * (y * y)
}

/// 2z + 2z = 4z.
theorem continued_fraction_periodic_int_two_z_add_two_z(z: Int) {
    Int.2 * z + Int.2 * z = Int.4 * z
} by {
    Int.2 * z + Int.2 * z = (Int.2 + Int.2) * z
    continued_fraction_periodic_int_two_add_two
    Int.2 + Int.2 = Int.4
    (Int.2 + Int.2) * z = Int.4 * z
    Int.2 * z + Int.2 * z = Int.4 * z
}

/// 3·(4z) = 6z + 6z.
theorem continued_fraction_periodic_int_three_times_four_z(z: Int) {
    Int.3 * (Int.4 * z) = Int.6 * z + Int.6 * z
} by {
    Int.4 * z = Int.2 * z + Int.2 * z
    continued_fraction_periodic_int_two_z_add_two_z(z)
    Int.2 * z + Int.2 * z = Int.4 * z
    Int.4 * z = Int.2 * z + Int.2 * z
    Int.3 * (Int.4 * z) = Int.3 * (Int.2 * z + Int.2 * z)
    Int.3 * (Int.2 * z + Int.2 * z) = Int.3 * (Int.2 * z) + Int.3 * (Int.2 * z)
    Int.3 * (Int.2 * z) = (Int.3 * Int.2) * z
    continued_fraction_periodic_int_three_mul_two
    Int.3 * Int.2 = Int.6
    (Int.3 * Int.2) * z = Int.6 * z
    Int.3 * (Int.2 * z) = Int.6 * z
    Int.3 * (Int.2 * z + Int.2 * z) = Int.6 * z + Int.6 * z
    Int.3 * (Int.4 * z) = Int.6 * z + Int.6 * z
}

/// 4x² - 3x² = x².
theorem continued_fraction_periodic_int_four_x2_sub_three_x2(x: Int) {
    Int.4 * (x * x) - Int.3 * (x * x) = x * x
} by {
    Int.4 * (x * x) - Int.3 * (x * x) = (Int.4 - Int.3) * (x * x)
    continued_fraction_periodic_int_four_sub_three
    Int.4 - Int.3 = Int.1
    (Int.4 - Int.3) * (x * x) = Int.1 * (x * x)
    Int.1 * (x * x) = x * x
    Int.4 * (x * x) - Int.3 * (x * x) = x * x
}

/// 9y² - 3·(4y²) = -3y².
theorem continued_fraction_periodic_int_nine_y2_sub_three_four_y2(y: Int) {
    Int.9 * (y * y) - Int.3 * (Int.4 * (y * y)) = -Int.3 * (y * y)
} by {
    Int.9 * (y * y) = Int.3 * (Int.3 * (y * y))
    Int.9 * (y * y) - Int.3 * (Int.4 * (y * y)) =
        Int.3 * (Int.3 * (y * y)) - Int.3 * (Int.4 * (y * y))
    Int.3 * (Int.3 * (y * y)) - Int.3 * (Int.4 * (y * y)) =
        Int.3 * (Int.3 * (y * y) - Int.4 * (y * y))
    Int.3 * (y * y) - Int.4 * (y * y) = (Int.3 - Int.4) * (y * y)
    continued_fraction_periodic_int_three_sub_four
    Int.3 - Int.4 = -Int.1
    (Int.3 - Int.4) * (y * y) = -Int.1 * (y * y)
    mul_neg_one_left(y * y)
    -Int.1 * (y * y) = -(y * y)
    Int.3 * (y * y) - Int.4 * (y * y) = -(y * y)
    Int.3 * (Int.3 * (y * y) - Int.4 * (y * y)) = Int.3 * (-(y * y))
    mul_neg_left(Int.3, y * y)
    Int.3 * (-(y * y)) = -(Int.3 * (y * y))
    Int.3 * (Int.3 * (y * y) - Int.4 * (y * y)) = -(Int.3 * (y * y))
    Int.9 * (y * y) - Int.3 * (Int.4 * (y * y)) = -(Int.3 * (y * y))
    Int.9 * (y * y) - Int.3 * (Int.4 * (y * y)) = -Int.3 * (y * y)
}

/// Reordering a four-term sum: a + b + c + d = a + d + b + c.
theorem continued_fraction_periodic_int_add_reorder_4(a: Int, b: Int, c: Int, d: Int) {
    a + b + c + d = a + d + b + c
}

/// Reordering a five-term sum: a + b + c + d + e = a + d + e + b + c.
theorem continued_fraction_periodic_int_add_reorder_5(a: Int, b: Int, c: Int, d: Int, e: Int) {
    a + b + c + d + e = a + d + e + b + c
}

/// Cancelling two common summands of a difference: (a + c + d) - (b + c + d) = a - b.
theorem continued_fraction_periodic_int_sub_cancel_two(a: Int, b: Int, c: Int, d: Int) {
    (a + c + d) - (b + c + d) = a - b
}

/// Composing a pair with the fundamental unit 2 + √3 preserves the Pell norm:
/// (2x + 3y)² - 3·(x + 2y)² = x² - 3·y².
theorem continued_fraction_periodic_compose_preserves_norm(x: Int, y: Int) {
    pell_norm(Int.3, Int.2 * x + Int.3 * y, x + Int.2 * y) = pell_norm(Int.3, x, y)
} by {
    pell_norm(Int.3, Int.2 * x + Int.3 * y, x + Int.2 * y) =
        (Int.2 * x + Int.3 * y) * (Int.2 * x + Int.3 * y) -
            Int.3 * ((x + Int.2 * y) * (x + Int.2 * y))
    (Int.2 * x + Int.3 * y) * (Int.2 * x + Int.3 * y) =
        (Int.2 * x + Int.3 * y) * (Int.2 * x) + (Int.2 * x + Int.3 * y) * (Int.3 * y)
    (Int.2 * x + Int.3 * y) * (Int.2 * x) = (Int.2 * x) * (Int.2 * x) + (Int.3 * y) * (Int.2 * x)
    continued_fraction_periodic_int_two_x_sq(x)
    (Int.2 * x) * (Int.2 * x) = Int.4 * (x * x)
    continued_fraction_periodic_int_three_y_two_x(x, y)
    (Int.3 * y) * (Int.2 * x) = Int.6 * (x * y)
    (Int.2 * x + Int.3 * y) * (Int.2 * x) = Int.4 * (x * x) + Int.6 * (x * y)
    (Int.2 * x + Int.3 * y) * (Int.3 * y) = (Int.2 * x) * (Int.3 * y) + (Int.3 * y) * (Int.3 * y)
    continued_fraction_periodic_int_two_x_three_y(x, y)
    (Int.2 * x) * (Int.3 * y) = Int.6 * (x * y)
    continued_fraction_periodic_int_three_y_sq(y)
    (Int.3 * y) * (Int.3 * y) = Int.9 * (y * y)
    (Int.2 * x + Int.3 * y) * (Int.3 * y) = Int.6 * (x * y) + Int.9 * (y * y)
    (Int.2 * x + Int.3 * y) * (Int.2 * x + Int.3 * y) =
        Int.4 * (x * x) + Int.6 * (x * y) + Int.6 * (x * y) + Int.9 * (y * y)
    (x + Int.2 * y) * (x + Int.2 * y) = (x + Int.2 * y) * x + (x + Int.2 * y) * (Int.2 * y)
    (x + Int.2 * y) * x = x * x + (Int.2 * y) * x
    (Int.2 * y) * x = Int.2 * (x * y)
    (x + Int.2 * y) * (Int.2 * y) = x * (Int.2 * y) + (Int.2 * y) * (Int.2 * y)
    x * (Int.2 * y) = Int.2 * (x * y)
    continued_fraction_periodic_int_two_y_sq(y)
    (Int.2 * y) * (Int.2 * y) = Int.4 * (y * y)
    (x + Int.2 * y) * (x + Int.2 * y) =
        x * x + Int.2 * (x * y) + Int.2 * (x * y) + Int.4 * (y * y)
    continued_fraction_periodic_int_two_z_add_two_z(x * y)
    Int.2 * (x * y) + Int.2 * (x * y) = Int.4 * (x * y)
    (x + Int.2 * y) * (x + Int.2 * y) = x * x + Int.4 * (x * y) + Int.4 * (y * y)
    Int.3 * ((x + Int.2 * y) * (x + Int.2 * y)) =
        Int.3 * (x * x + Int.4 * (x * y) + Int.4 * (y * y))
    Int.3 * (x * x + Int.4 * (x * y) + Int.4 * (y * y)) =
        Int.3 * (x * x) + Int.3 * (Int.4 * (x * y)) + Int.3 * (Int.4 * (y * y))
    continued_fraction_periodic_int_three_times_four_z(x * y)
    Int.3 * (Int.4 * (x * y)) = Int.6 * (x * y) + Int.6 * (x * y)
    continued_fraction_periodic_int_three_times_four_z(y * y)
    Int.3 * (Int.4 * (y * y)) = Int.6 * (y * y) + Int.6 * (y * y)
    Int.3 * (x * x + Int.4 * (x * y) + Int.4 * (y * y)) =
        Int.3 * (x * x) + Int.6 * (x * y) + Int.6 * (x * y) + Int.6 * (y * y) + Int.6 * (y * y)
    pell_norm(Int.3, Int.2 * x + Int.3 * y, x + Int.2 * y) =
        (Int.4 * (x * x) + Int.6 * (x * y) + Int.6 * (x * y) + Int.9 * (y * y)) -
            (Int.3 * (x * x) + Int.6 * (x * y) + Int.6 * (x * y) + Int.6 * (y * y) + Int.6 * (y * y))
    continued_fraction_periodic_int_add_reorder_4(
        Int.4 * (x * x), Int.6 * (x * y), Int.6 * (x * y), Int.9 * (y * y))
    Int.4 * (x * x) + Int.6 * (x * y) + Int.6 * (x * y) + Int.9 * (y * y) =
        Int.4 * (x * x) + Int.9 * (y * y) + Int.6 * (x * y) + Int.6 * (x * y)
    continued_fraction_periodic_int_add_reorder_5(
        Int.3 * (x * x), Int.6 * (x * y), Int.6 * (x * y), Int.6 * (y * y), Int.6 * (y * y))
    Int.3 * (x * x) + Int.6 * (x * y) + Int.6 * (x * y) + Int.6 * (y * y) + Int.6 * (y * y) =
        Int.3 * (x * x) + Int.6 * (y * y) + Int.6 * (y * y) + Int.6 * (x * y) + Int.6 * (x * y)
    pell_norm(Int.3, Int.2 * x + Int.3 * y, x + Int.2 * y) =
        (Int.4 * (x * x) + Int.9 * (y * y) + Int.6 * (x * y) + Int.6 * (x * y)) -
            (Int.3 * (x * x) + Int.6 * (y * y) + Int.6 * (y * y) + Int.6 * (x * y) + Int.6 * (x * y))
    continued_fraction_periodic_int_sub_cancel_two(
        Int.4 * (x * x) + Int.9 * (y * y),
        Int.3 * (x * x) + Int.6 * (y * y) + Int.6 * (y * y),
        Int.6 * (x * y), Int.6 * (x * y))
    (Int.4 * (x * x) + Int.9 * (y * y) + Int.6 * (x * y) + Int.6 * (x * y)) -
            (Int.3 * (x * x) + Int.6 * (y * y) + Int.6 * (y * y) + Int.6 * (x * y) + Int.6 * (x * y)) =
        (Int.4 * (x * x) + Int.9 * (y * y)) -
            (Int.3 * (x * x) + Int.6 * (y * y) + Int.6 * (y * y))
    pell_norm(Int.3, Int.2 * x + Int.3 * y, x + Int.2 * y) =
        (Int.4 * (x * x) + Int.9 * (y * y)) - (Int.3 * (x * x) + Int.6 * (y * y) + Int.6 * (y * y))
    pell_rearrange_sub_sum(
        Int.4 * (x * x) + Int.9 * (y * y), Int.3 * (x * x), Int.6 * (y * y) + Int.6 * (y * y))
    (Int.4 * (x * x) + Int.9 * (y * y)) - (Int.3 * (x * x) + (Int.6 * (y * y) + Int.6 * (y * y))) =
        (Int.4 * (x * x) + Int.9 * (y * y)) - Int.3 * (x * x) - (Int.6 * (y * y) + Int.6 * (y * y))
    pell_rearrange_flatten(
        Int.4 * (x * x), Int.9 * (y * y), Int.3 * (x * x), Int.6 * (y * y) + Int.6 * (y * y))
    (Int.4 * (x * x) + Int.9 * (y * y)) - Int.3 * (x * x) - (Int.6 * (y * y) + Int.6 * (y * y)) =
        Int.4 * (x * x) + Int.9 * (y * y) - Int.3 * (x * x) - (Int.6 * (y * y) + Int.6 * (y * y))
    pell_norm(Int.3, Int.2 * x + Int.3 * y, x + Int.2 * y) =
        Int.4 * (x * x) + Int.9 * (y * y) - Int.3 * (x * x) - (Int.6 * (y * y) + Int.6 * (y * y))
    cf_pell_sub_pair_identity(
        Int.4 * (x * x), Int.9 * (y * y), Int.3 * (x * x), Int.6 * (y * y) + Int.6 * (y * y))
    Int.4 * (x * x) + Int.9 * (y * y) - Int.3 * (x * x) - (Int.6 * (y * y) + Int.6 * (y * y)) =
        (Int.4 * (x * x) - Int.3 * (x * x)) + (Int.9 * (y * y) - (Int.6 * (y * y) + Int.6 * (y * y)))
    continued_fraction_periodic_int_four_x2_sub_three_x2(x)
    Int.4 * (x * x) - Int.3 * (x * x) = x * x
    Int.9 * (y * y) - (Int.6 * (y * y) + Int.6 * (y * y)) =
        Int.9 * (y * y) - Int.6 * (y * y) - Int.6 * (y * y)
    Int.6 * (y * y) + Int.6 * (y * y) = Int.3 * (Int.4 * (y * y))
    continued_fraction_periodic_int_three_times_four_z(y * y)
    Int.9 * (y * y) - Int.6 * (y * y) - Int.6 * (y * y) =
        Int.9 * (y * y) - Int.3 * (Int.4 * (y * y))
    continued_fraction_periodic_int_nine_y2_sub_three_four_y2(y)
    Int.9 * (y * y) - Int.3 * (Int.4 * (y * y)) = -Int.3 * (y * y)
    Int.9 * (y * y) - (Int.6 * (y * y) + Int.6 * (y * y)) = -Int.3 * (y * y)
    (Int.4 * (x * x) - Int.3 * (x * x)) + (Int.9 * (y * y) - (Int.6 * (y * y) + Int.6 * (y * y))) =
        x * x + -Int.3 * (y * y)
    pell_norm(Int.3, Int.2 * x + Int.3 * y, x + Int.2 * y) = x * x + -Int.3 * (y * y)
    pell_norm(Int.3, x, y) = x * x - Int.3 * (y * y)
    x * x - Int.3 * (y * y) = x * x + -Int.3 * (y * y)
    pell_norm(Int.3, x, y) = x * x + -Int.3 * (y * y)
    pell_norm(Int.3, Int.2 * x + Int.3 * y, x + Int.2 * y) = pell_norm(Int.3, x, y)
}

/// The zeroth convergent of √3 has norm minus two: p_0² - 3·q_0² = -2.
theorem continued_fraction_periodic_sqrt_three_convergent_norm_zero {
    pell_norm(Int.3,
        Int.from_nat(continued_fraction_convergent_numerator(
            sqrt_three_continued_fraction_coefficients, Nat.0)),
        Int.from_nat(continued_fraction_convergent_denominator(
            sqrt_three_continued_fraction_coefficients, Nat.0))) = -Int.2
} by {
    continued_fraction_periodic_sqrt_three_convergent_numerator_zero
    continued_fraction_convergent_numerator(
        sqrt_three_continued_fraction_coefficients, Nat.0) = Nat.1
    continued_fraction_periodic_sqrt_three_convergent_denominator_zero
    continued_fraction_convergent_denominator(
        sqrt_three_continued_fraction_coefficients, Nat.0) = Nat.1
    Int.from_nat(Nat.1) = Int.1
    pell_norm(Int.3,
        Int.from_nat(continued_fraction_convergent_numerator(
            sqrt_three_continued_fraction_coefficients, Nat.0)),
        Int.from_nat(continued_fraction_convergent_denominator(
            sqrt_three_continued_fraction_coefficients, Nat.0))) =
        pell_norm(Int.3, Int.1, Int.1)
    pell_norm(Int.3, Int.1, Int.1) = Int.1 * Int.1 - Int.3 * (Int.1 * Int.1)
    Int.1 * Int.1 = Int.1
    Int.3 * (Int.1 * Int.1) = Int.3
    continued_fraction_periodic_int_one_sub_three
    Int.1 - Int.3 = -Int.2
    pell_norm(Int.3, Int.1, Int.1) = -Int.2
    pell_norm(Int.3,
        Int.from_nat(continued_fraction_convergent_numerator(
            sqrt_three_continued_fraction_coefficients, Nat.0)),
        Int.from_nat(continued_fraction_convergent_denominator(
            sqrt_three_continued_fraction_coefficients, Nat.0))) = -Int.2
}

/// The first convergent of √3 has norm one: p_1² - 3·q_1² = 1.
theorem continued_fraction_periodic_sqrt_three_convergent_norm_one {
    pell_norm(Int.3,
        Int.from_nat(continued_fraction_convergent_numerator(
            sqrt_three_continued_fraction_coefficients, Nat.1)),
        Int.from_nat(continued_fraction_convergent_denominator(
            sqrt_three_continued_fraction_coefficients, Nat.1))) = Int.1
} by {
    continued_fraction_periodic_sqrt_three_convergent_numerator_one
    continued_fraction_convergent_numerator(
        sqrt_three_continued_fraction_coefficients, Nat.1) = Nat.2
    continued_fraction_periodic_sqrt_three_convergent_denominator_one
    continued_fraction_convergent_denominator(
        sqrt_three_continued_fraction_coefficients, Nat.1) = Nat.1
    Int.from_nat(Nat.2) = Int.2
    Int.from_nat(Nat.1) = Int.1
    pell_norm(Int.3,
        Int.from_nat(continued_fraction_convergent_numerator(
            sqrt_three_continued_fraction_coefficients, Nat.1)),
        Int.from_nat(continued_fraction_convergent_denominator(
            sqrt_three_continued_fraction_coefficients, Nat.1))) =
        pell_norm(Int.3, Int.2, Int.1)
    pell_norm(Int.3, Int.2, Int.1) = Int.2 * Int.2 - Int.3 * (Int.1 * Int.1)
    mul_from_nat(Nat.2, Nat.2)
    Int.2 * Int.2 = Int.4
    Int.1 * Int.1 = Int.1
    Int.3 * (Int.1 * Int.1) = Int.3
    continued_fraction_periodic_int_four_sub_three
    Int.4 - Int.3 = Int.1
    pell_norm(Int.3, Int.2, Int.1) = Int.1
    pell_norm(Int.3,
        Int.from_nat(continued_fraction_convergent_numerator(
            sqrt_three_continued_fraction_coefficients, Nat.1)),
        Int.from_nat(continued_fraction_convergent_denominator(
            sqrt_three_continued_fraction_coefficients, Nat.1))) = Int.1
}

/// The second convergent of √3 has norm minus two: p_2² - 3·q_2² = -2.
theorem continued_fraction_periodic_sqrt_three_convergent_norm_two {
    pell_norm(Int.3,
        Int.from_nat(continued_fraction_convergent_numerator(
            sqrt_three_continued_fraction_coefficients, Nat.2)),
        Int.from_nat(continued_fraction_convergent_denominator(
            sqrt_three_continued_fraction_coefficients, Nat.2))) = -Int.2
} by {
    continued_fraction_periodic_sqrt_three_convergent_numerator_two
    continued_fraction_convergent_numerator(
        sqrt_three_continued_fraction_coefficients, Nat.2) = Nat.5
    continued_fraction_periodic_sqrt_three_convergent_denominator_two
    continued_fraction_convergent_denominator(
        sqrt_three_continued_fraction_coefficients, Nat.2) = Nat.3
    Int.from_nat(Nat.5) = Int.5
    Int.from_nat(Nat.3) = Int.3
    pell_norm(Int.3,
        Int.from_nat(continued_fraction_convergent_numerator(
            sqrt_three_continued_fraction_coefficients, Nat.2)),
        Int.from_nat(continued_fraction_convergent_denominator(
            sqrt_three_continued_fraction_coefficients, Nat.2))) =
        pell_norm(Int.3, Int.5, Int.3)
    mul_one_right(Int.2)
    Int.2 * Int.1 = Int.2
    mul_one_right(Int.3)
    Int.3 * Int.1 = Int.3
    add_from_nat(Nat.2, Nat.3)
    Int.from_nat(Nat.2) + Int.from_nat(Nat.3) = Int.from_nat(Nat.2 + Nat.3)
    Nat.2 + Nat.3 = Nat.5
    Int.from_nat(Nat.2) + Int.from_nat(Nat.3) = Int.from_nat(Nat.5)
    Int.from_nat(Nat.2) = Int.2
    Int.from_nat(Nat.3) = Int.3
    Int.from_nat(Nat.5) = Int.5
    Int.2 + Int.3 = Int.5
    Int.2 * Int.1 + Int.3 * Int.1 = Int.5
    mul_one_right(Int.2)
    Int.2 * Int.1 = Int.2
    add_from_nat(Nat.1, Nat.2)
    Int.from_nat(Nat.1) + Int.from_nat(Nat.2) = Int.from_nat(Nat.1 + Nat.2)
    Nat.1 + Nat.2 = Nat.3
    Int.from_nat(Nat.1) + Int.from_nat(Nat.2) = Int.from_nat(Nat.3)
    Int.from_nat(Nat.1) = Int.1
    Int.from_nat(Nat.2) = Int.2
    Int.from_nat(Nat.3) = Int.3
    Int.1 + Int.2 = Int.3
    Int.1 + Int.2 * Int.1 = Int.3
    pell_norm(Int.3, Int.5, Int.3) =
        pell_norm(Int.3, Int.2 * Int.1 + Int.3 * Int.1, Int.1 + Int.2 * Int.1)
    continued_fraction_periodic_compose_preserves_norm(Int.1, Int.1)
    pell_norm(Int.3, Int.2 * Int.1 + Int.3 * Int.1, Int.1 + Int.2 * Int.1) =
        pell_norm(Int.3, Int.1, Int.1)
    continued_fraction_periodic_sqrt_three_convergent_norm_zero
    pell_norm(Int.3,
        Int.from_nat(continued_fraction_convergent_numerator(
            sqrt_three_continued_fraction_coefficients, Nat.0)),
        Int.from_nat(continued_fraction_convergent_denominator(
            sqrt_three_continued_fraction_coefficients, Nat.0))) = -Int.2
    continued_fraction_periodic_sqrt_three_convergent_numerator_zero
    continued_fraction_convergent_numerator(
        sqrt_three_continued_fraction_coefficients, Nat.0) = Nat.1
    continued_fraction_periodic_sqrt_three_convergent_denominator_zero
    continued_fraction_convergent_denominator(
        sqrt_three_continued_fraction_coefficients, Nat.0) = Nat.1
    Int.from_nat(Nat.1) = Int.1
    pell_norm(Int.3, Int.1, Int.1) = -Int.2
    pell_norm(Int.3, Int.5, Int.3) = -Int.2
    pell_norm(Int.3,
        Int.from_nat(continued_fraction_convergent_numerator(
            sqrt_three_continued_fraction_coefficients, Nat.2)),
        Int.from_nat(continued_fraction_convergent_denominator(
            sqrt_three_continued_fraction_coefficients, Nat.2))) = -Int.2
}

/// The third convergent of √3 has norm one: p_3² - 3·q_3² = 1.
theorem continued_fraction_periodic_sqrt_three_convergent_norm_three {
    pell_norm(Int.3,
        Int.from_nat(continued_fraction_convergent_numerator(
            sqrt_three_continued_fraction_coefficients, Nat.3)),
        Int.from_nat(continued_fraction_convergent_denominator(
            sqrt_three_continued_fraction_coefficients, Nat.3))) = Int.1
} by {
    continued_fraction_periodic_sqrt_three_convergent_numerator_three
    continued_fraction_convergent_numerator(
        sqrt_three_continued_fraction_coefficients, Nat.3) = Nat.7
    continued_fraction_periodic_sqrt_three_convergent_denominator_three
    continued_fraction_convergent_denominator(
        sqrt_three_continued_fraction_coefficients, Nat.3) = Nat.4
    Int.from_nat(Nat.7) = Int.7
    Int.from_nat(Nat.4) = Int.4
    pell_norm(Int.3,
        Int.from_nat(continued_fraction_convergent_numerator(
            sqrt_three_continued_fraction_coefficients, Nat.3)),
        Int.from_nat(continued_fraction_convergent_denominator(
            sqrt_three_continued_fraction_coefficients, Nat.3))) =
        pell_norm(Int.3, Int.7, Int.4)
    continued_fraction_periodic_int_two_mul_two
    Int.2 * Int.2 = Int.4
    mul_one_right(Int.3)
    Int.3 * Int.1 = Int.3
    add_from_nat(Nat.4, Nat.3)
    Int.from_nat(Nat.4) + Int.from_nat(Nat.3) = Int.from_nat(Nat.4 + Nat.3)
    Nat.4 + Nat.3 = Nat.7
    Int.from_nat(Nat.4) + Int.from_nat(Nat.3) = Int.from_nat(Nat.7)
    Int.from_nat(Nat.4) = Int.4
    Int.from_nat(Nat.3) = Int.3
    Int.from_nat(Nat.7) = Int.7
    Int.4 + Int.3 = Int.7
    Int.2 * Int.2 + Int.3 * Int.1 = Int.7
    mul_one_right(Int.2)
    Int.2 * Int.1 = Int.2
    add_from_nat(Nat.2, Nat.2)
    Int.from_nat(Nat.2) + Int.from_nat(Nat.2) = Int.from_nat(Nat.2 + Nat.2)
    Nat.2 + Nat.2 = Nat.4
    Int.from_nat(Nat.2) + Int.from_nat(Nat.2) = Int.from_nat(Nat.4)
    Int.from_nat(Nat.2) = Int.2
    Int.from_nat(Nat.4) = Int.4
    Int.2 + Int.2 = Int.4
    Int.2 + Int.2 * Int.1 = Int.4
    pell_norm(Int.3, Int.7, Int.4) =
        pell_norm(Int.3, Int.2 * Int.2 + Int.3 * Int.1, Int.2 + Int.2 * Int.1)
    continued_fraction_periodic_compose_preserves_norm(Int.2, Int.1)
    pell_norm(Int.3, Int.2 * Int.2 + Int.3 * Int.1, Int.2 + Int.2 * Int.1) =
        pell_norm(Int.3, Int.2, Int.1)
    continued_fraction_periodic_sqrt_three_convergent_norm_one
    pell_norm(Int.3,
        Int.from_nat(continued_fraction_convergent_numerator(
            sqrt_three_continued_fraction_coefficients, Nat.1)),
        Int.from_nat(continued_fraction_convergent_denominator(
            sqrt_three_continued_fraction_coefficients, Nat.1))) = Int.1
    continued_fraction_periodic_sqrt_three_convergent_numerator_one
    continued_fraction_convergent_numerator(
        sqrt_three_continued_fraction_coefficients, Nat.1) = Nat.2
    continued_fraction_periodic_sqrt_three_convergent_denominator_one
    continued_fraction_convergent_denominator(
        sqrt_three_continued_fraction_coefficients, Nat.1) = Nat.1
    Int.from_nat(Nat.2) = Int.2
    Int.from_nat(Nat.1) = Int.1
    pell_norm(Int.3, Int.2, Int.1) = Int.1
    pell_norm(Int.3, Int.7, Int.4) = Int.1
    pell_norm(Int.3,
        Int.from_nat(continued_fraction_convergent_numerator(
            sqrt_three_continued_fraction_coefficients, Nat.3)),
        Int.from_nat(continued_fraction_convergent_denominator(
            sqrt_three_continued_fraction_coefficients, Nat.3))) = Int.1
}

/// The first four norms of the convergents of √3 alternate between minus two
/// and one: -2, 1, -2, 1.
theorem continued_fraction_periodic_sqrt_three_first_norms {
    pell_norm(Int.3,
        Int.from_nat(continued_fraction_convergent_numerator(
            sqrt_three_continued_fraction_coefficients, Nat.0)),
        Int.from_nat(continued_fraction_convergent_denominator(
            sqrt_three_continued_fraction_coefficients, Nat.0))) = -Int.2 and
    pell_norm(Int.3,
        Int.from_nat(continued_fraction_convergent_numerator(
            sqrt_three_continued_fraction_coefficients, Nat.1)),
        Int.from_nat(continued_fraction_convergent_denominator(
            sqrt_three_continued_fraction_coefficients, Nat.1))) = Int.1 and
    pell_norm(Int.3,
        Int.from_nat(continued_fraction_convergent_numerator(
            sqrt_three_continued_fraction_coefficients, Nat.2)),
        Int.from_nat(continued_fraction_convergent_denominator(
            sqrt_three_continued_fraction_coefficients, Nat.2))) = -Int.2 and
    pell_norm(Int.3,
        Int.from_nat(continued_fraction_convergent_numerator(
            sqrt_three_continued_fraction_coefficients, Nat.3)),
        Int.from_nat(continued_fraction_convergent_denominator(
            sqrt_three_continued_fraction_coefficients, Nat.3))) = Int.1
} by {
    continued_fraction_periodic_sqrt_three_convergent_norm_zero
    pell_norm(Int.3,
        Int.from_nat(continued_fraction_convergent_numerator(
            sqrt_three_continued_fraction_coefficients, Nat.0)),
        Int.from_nat(continued_fraction_convergent_denominator(
            sqrt_three_continued_fraction_coefficients, Nat.0))) = -Int.2
    continued_fraction_periodic_sqrt_three_convergent_norm_one
    pell_norm(Int.3,
        Int.from_nat(continued_fraction_convergent_numerator(
            sqrt_three_continued_fraction_coefficients, Nat.1)),
        Int.from_nat(continued_fraction_convergent_denominator(
            sqrt_three_continued_fraction_coefficients, Nat.1))) = Int.1
    continued_fraction_periodic_sqrt_three_convergent_norm_two
    pell_norm(Int.3,
        Int.from_nat(continued_fraction_convergent_numerator(
            sqrt_three_continued_fraction_coefficients, Nat.2)),
        Int.from_nat(continued_fraction_convergent_denominator(
            sqrt_three_continued_fraction_coefficients, Nat.2))) = -Int.2
    continued_fraction_periodic_sqrt_three_convergent_norm_three
    pell_norm(Int.3,
        Int.from_nat(continued_fraction_convergent_numerator(
            sqrt_three_continued_fraction_coefficients, Nat.3)),
        Int.from_nat(continued_fraction_convergent_denominator(
            sqrt_three_continued_fraction_coefficients, Nat.3))) = Int.1
    pell_norm(Int.3,
        Int.from_nat(continued_fraction_convergent_numerator(
            sqrt_three_continued_fraction_coefficients, Nat.0)),
        Int.from_nat(continued_fraction_convergent_denominator(
            sqrt_three_continued_fraction_coefficients, Nat.0))) = -Int.2 and
    pell_norm(Int.3,
        Int.from_nat(continued_fraction_convergent_numerator(
            sqrt_three_continued_fraction_coefficients, Nat.1)),
        Int.from_nat(continued_fraction_convergent_denominator(
            sqrt_three_continued_fraction_coefficients, Nat.1))) = Int.1 and
    pell_norm(Int.3,
        Int.from_nat(continued_fraction_convergent_numerator(
            sqrt_three_continued_fraction_coefficients, Nat.2)),
        Int.from_nat(continued_fraction_convergent_denominator(
            sqrt_three_continued_fraction_coefficients, Nat.2))) = -Int.2 and
    pell_norm(Int.3,
        Int.from_nat(continued_fraction_convergent_numerator(
            sqrt_three_continued_fraction_coefficients, Nat.3)),
        Int.from_nat(continued_fraction_convergent_denominator(
            sqrt_three_continued_fraction_coefficients, Nat.3))) = Int.1
}

// The general norm pattern for the convergents of √3 — the odd-indexed
// convergents all have norm one:
//
// theorem continued_fraction_periodic_sqrt_three_odd_convergent_norm_one(k: Nat) {
//     pell_norm(Int.3,
//         Int.from_nat(continued_fraction_convergent_numerator(
//             sqrt_three_continued_fraction_coefficients, (Nat.2 * k).suc)),
//         Int.from_nat(continued_fraction_convergent_denominator(
//             sqrt_three_continued_fraction_coefficients, (Nat.2 * k).suc))) = Int.1
// }
//
// and the even-indexed convergents all have norm minus two — is proved for the
// first four convergents above.  The general statement needs the composition
// step of the fundamental unit 2 + √3 (p_{2k+3} = 2·p_{2k+1} + 3·q_{2k+1},
// q_{2k+3} = p_{2k+1} + 2·q_{2k+1}), which preserves the norm
// (2x + 3y)² - 3·(x + 2y)² = x² - 3·y², exactly as cf_pell.ac proves the
// composition step of 1 + √2 for the convergents of √2; it is left for future
// work.

// The classical estimate for the convergents of √d, stated in pell.ac: for a
// convergent p_n/q_n of √d,
//
//     |p_n² - d·q_n²| < 2·√d + 1.
//
// Its proof requires the convergence of the continued fraction of √d to √d as
// a real number, which is not yet formalized.  For d = 2 the sharp value is
// proved exactly above: |p_n² - 2·q_n²| = 1.

// ============================================================================
// Section 5: Lagrange's theorem and its converse
// ============================================================================

/// True when a coefficient sequence is eventually periodic: from some offset
/// on, the sequence repeats with a fixed positive period.
define eventually_periodic_sequence(coefficients: Nat -> Nat) -> Bool {
    exists(offset: Nat, period: Nat) {
        Nat.0 < period and
        forall(n: Nat) {
            coefficients(offset + n) = coefficients(offset + n + period)
        }
    }
}

/// True when a real number is rational.
define is_rational_real(x: Real) -> Bool {
    exists(p: Rat) {
        x = Real.from_rat(p)
    }
}

/// True when a real number is a quadratic irrational: it is irrational and the
/// root of a quadratic polynomial with integer coefficients.
define is_quadratic_irrational(x: Real) -> Bool {
    not is_rational_real(x) and
    exists(a: Int, b: Int, c: Int) {
        a != Int.0 and
        Real.from_int(a) * x * x + Real.from_int(b) * x + Real.from_int(c) = Real.0
    }
}

/// The coefficient sequence of √2 is eventually periodic: it repeats with
/// period one from the second coefficient on.
theorem continued_fraction_periodic_sqrt_two_eventually_periodic {
    eventually_periodic_sequence(sqrt_two_continued_fraction_coefficients)
} by {
    Nat.0 < Nat.1
    forall(n: Nat) {
        continued_fraction_periodic_sqrt_two_period_one(n)
        sqrt_two_continued_fraction_coefficients(n.suc) =
            sqrt_two_continued_fraction_coefficients(n.suc.suc)
        add_comm(Nat.1, n)
        Nat.1 + n = n + Nat.1
        add_one_right(n)
        n + Nat.1 = n.suc
        Nat.1 + n = n.suc
        Nat.1 + n + Nat.1 = n.suc.suc
        sqrt_two_continued_fraction_coefficients(Nat.1 + n) =
            sqrt_two_continued_fraction_coefficients(Nat.1 + n + Nat.1)
    }
    exists(offset: Nat, period: Nat) {
        Nat.0 < period and
        forall(n: Nat) {
            sqrt_two_continued_fraction_coefficients(offset + n) =
                sqrt_two_continued_fraction_coefficients(offset + n + period)
        }
    }
    eventually_periodic_sequence(sqrt_two_continued_fraction_coefficients)
}

// The classical theorem of Lagrange: the continued fraction of a quadratic
// irrational is eventually periodic.  In the library's formulation, for a
// coefficient sequence with a positive tail whose real limit is a quadratic
// irrational:
//
// theorem continued_fraction_periodic_lagrange(coefficients: Nat -> Nat) {
//     positive_continued_fraction_sequence_tail(coefficients) and
//         is_quadratic_irrational(continued_fraction_real_limit(coefficients))
//         implies eventually_periodic_sequence(coefficients)
// }
//
// This is the deep theorem of the theory of continued fractions.  Its proof
// uses the finiteness of the possible remainders (P_k + √D)/Q_k of the
// Euclidean algorithm for √D, which forces a repetition of the state and hence
// of the partial quotients; it is left for future work.  The two examples of
// this file — √2 = [1; 2, 2, 2, ...] and √3 = [1; 1, 2, 1, 2, ...] — exhibit
// the phenomenon concretely, and the eventual periodicity of the √2 sequence
// is proved above in the general form eventually_periodic_sequence.

// The converse of Lagrange's theorem: a number with an eventually periodic
// continued fraction is a quadratic irrational.  In the library's
// formulation, for a coefficient sequence with a positive tail:
//
// theorem continued_fraction_periodic_converse(coefficients: Nat -> Nat) {
//     positive_continued_fraction_sequence_tail(coefficients) and
//         eventually_periodic_sequence(coefficients)
//         implies is_quadratic_irrational(continued_fraction_real_limit(coefficients))
// }
//
// The converse is classical and elementary: a periodic continued fraction
// satisfies a quadratic equation, obtained by the self-substitution of the
// repeating block into the defining equation x = a_0 + 1/(a_1 + 1/(...));
// it is left for future work.
