from number_theory.congruence import Nat, congr_mod_mul, congr_mod_trans, congr_mod_symm, congr_mod_refl
from number_theory.coprime import coprime_one_left, coprime_comm, coprime_mod_iff, coprime_mul, coprime_mul_imp_right
from number_theory.bezout import nat_bezout
from number_theory.congr_int import nat_congr_mod_iff_int_mod_rel, mul_from_nat
from int import Int
from data.int.int_residue import int_has_nat_residue
from zmod import int_mod_rel, int_mod_rel_mul_compatible, int_mod_rel_is_equivalence
from data.basic.relation_basic import is_transitive
from algebra.ring.ring import mul_neg_left
numerals Int

from nat import gcd_zero_right

/// Modular inverse on the integers: when a is coprime to n on Nat, there
/// exists an integer b with b * Int.from_nat(a) congruent to 1 modulo n.
theorem int_modular_inverse_exists(a: Nat, n: Nat) {
    a.coprime(n) implies
        exists(b: Int) { int_mod_rel(n, b * Int.from_nat(a), Int.1) }
} by {
    if a.coprime(n) {
        a.gcd(n) = Nat.1
        nat_bezout(a, n)
        let (x: Int, y: Int) satisfy {
            x * Int.from_nat(a) + y * Int.from_nat(n) = Int.from_nat(a.gcd(n))
        }
        Int.from_nat(a.gcd(n)) = Int.from_nat(Nat.1)
        Int.from_nat(Nat.1) = Int.1
        x * Int.from_nat(a) + y * Int.from_nat(n) = Int.1
        x * Int.from_nat(a) - Int.1 = -(y * Int.from_nat(n))
        mul_neg_left(y, Int.from_nat(n))
        (-y) * Int.from_nat(n) = -(y * Int.from_nat(n))
        x * Int.from_nat(a) - Int.1 = (-y) * Int.from_nat(n)
        Int.from_nat(n) * (-y) = (-y) * Int.from_nat(n)
        Int.from_nat(n) * (-y) = x * Int.from_nat(a) - Int.1
        Int.from_nat(n).divides(x * Int.from_nat(a) - Int.1)
        int_mod_rel(n, x * Int.from_nat(a), Int.1)
        exists(b: Int) { int_mod_rel(n, b * Int.from_nat(a), Int.1) }
    }
}

/// Modular inverse on the naturals when n is positive.
theorem nat_modular_inverse_exists_pos(a: Nat, n: Nat) {
    n != Nat.0 and a.coprime(n) implies exists(b: Nat) { (a * b).congr_mod(Nat.1, n) }
} by {
    if n != Nat.0 and a.coprime(n) {
        int_modular_inverse_exists(a, n)
        let bi: Int satisfy { int_mod_rel(n, bi * Int.from_nat(a), Int.1) }
        int_has_nat_residue(bi, n)
        let r: Nat satisfy { int_mod_rel(n, bi, Int.from_nat(r)) }
        int_mod_rel_is_equivalence(n)
        int_mod_rel(n, Int.from_nat(a), Int.from_nat(a))
        int_mod_rel_mul_compatible(n, bi, Int.from_nat(r), Int.from_nat(a), Int.from_nat(a))
        int_mod_rel(n, bi * Int.from_nat(a), Int.from_nat(r) * Int.from_nat(a))
        is_transitive(int_mod_rel(n))
        int_mod_rel(n, Int.from_nat(r) * Int.from_nat(a), bi * Int.from_nat(a))
        int_mod_rel(n, Int.from_nat(r) * Int.from_nat(a), Int.1)
        mul_from_nat(r, a)
        Int.from_nat(r) * Int.from_nat(a) = Int.from_nat(r * a)
        int_mod_rel(n, Int.from_nat(r * a), Int.1)
        Int.from_nat(Nat.1) = Int.1
        int_mod_rel(n, Int.from_nat(r * a), Int.from_nat(Nat.1))
        nat_congr_mod_iff_int_mod_rel(r * a, Nat.1, n)
        (r * a).congr_mod(Nat.1, n)
        r * a = a * r
        (a * r).congr_mod(Nat.1, n)
        exists(b: Nat) { (a * b).congr_mod(Nat.1, n) }
    }
}

/// Modular inverse on the naturals when n is zero: coprime to 0 forces a = 1.
theorem nat_modular_inverse_exists_zero(a: Nat) {
    a.coprime(Nat.0) implies exists(b: Nat) { (a * b).congr_mod(Nat.1, Nat.0) }
} by {
    if a.coprime(Nat.0) {
        a.gcd(Nat.0) = Nat.1
        gcd_zero_right(a)
        a = Nat.1
        (a * Nat.1).congr_mod(Nat.1, Nat.0)
    }
}

/// Modular inverse on the naturals: when a is coprime to n there exists a
/// Nat b with a * b congruent to 1 modulo n.
theorem nat_modular_inverse_exists(a: Nat, n: Nat) {
    a.coprime(n) implies exists(b: Nat) { (a * b).congr_mod(Nat.1, n) }
} by {
    n = Nat.0 or n != Nat.0
    if n = Nat.0 {
        if a.coprime(n) {
            a.coprime(Nat.0)
            nat_modular_inverse_exists_zero(a)
            let b0: Nat satisfy { (a * b0).congr_mod(Nat.1, Nat.0) }
            (a * b0).congr_mod(Nat.1, n)
        }
    } else {
        if a.coprime(n) {
            nat_modular_inverse_exists_pos(a, n)
        }
    }
}

attributes Nat {
    /// True when b is a modular inverse of self modulo n: their product is
    /// congruent to one modulo n.
    define is_mod_inv(self, b: Nat, n: Nat) -> Bool {
        (self * b).congr_mod(Nat.1, n)
    }
}

/// Step 1 of mod_inv_unique: scale (a * b2) ≡ 1 by b1 on the left to obtain
/// b1 * (a * b2) ≡ b1 (mod n).
theorem mod_inv_unique_left_scale(a: Nat, n: Nat, b1: Nat, b2: Nat) {
    (a * b2).congr_mod(Nat.1, n) implies (b1 * (a * b2)).congr_mod(b1, n)
} by {
    if (a * b2).congr_mod(Nat.1, n) {
        congr_mod_refl(b1, n)
        b1.congr_mod(b1, n)
        congr_mod_mul(b1, a * b2, b1, Nat.1, n)
    }
}

/// Step 2 of mod_inv_unique: scale (a * b1) ≡ 1 by b2 on the right to obtain
/// (a * b1) * b2 ≡ b2 (mod n).
theorem mod_inv_unique_right_scale(a: Nat, n: Nat, b1: Nat, b2: Nat) {
    (a * b1).congr_mod(Nat.1, n) implies ((a * b1) * b2).congr_mod(b2, n)
} by {
    if (a * b1).congr_mod(Nat.1, n) {
        congr_mod_refl(b2, n)
        b2.congr_mod(b2, n)
        congr_mod_mul(a * b1, b2, Nat.1, b2, n)
    }
}

/// Helper for mod_inv_unique_congr: rewrite b1 * (a * b2) into (a * b1) * b2.
theorem mod_inv_assoc_swap(a: Nat, b1: Nat, b2: Nat) {
    b1 * (a * b2) = (a * b1) * b2
} by {
    b1 * (a * b2) = (b1 * a) * b2
    b1 * a = a * b1
}

/// Modular inverses up to congruence: if a*b1 and a*b2 are both ≡ 1 mod n,
/// then b1 ≡ b2 mod n. The key cancellation argument used in
/// `mod_inv_unique`.
theorem mod_inv_unique_congr(a: Nat, n: Nat, b1: Nat, b2: Nat) {
    (a * b1).congr_mod(Nat.1, n) and (a * b2).congr_mod(Nat.1, n)
        implies b1.congr_mod(b2, n)
} by {
    if (a * b1).congr_mod(Nat.1, n) and (a * b2).congr_mod(Nat.1, n) {
        mod_inv_unique_left_scale(a, n, b1, b2)
        let lhs: Nat = b1 * (a * b2)
        lhs.congr_mod(b1, n)
        mod_inv_assoc_swap(a, b1, b2)
        let prod: Nat = (a * b1) * b2
        lhs = prod
        prod.congr_mod(b1, n)
        mod_inv_unique_right_scale(a, n, b1, b2)
        prod.congr_mod(b2, n)
        // Both b1 and b2 are equal to prod mod n; use congr_mod definition.
        prod.mod(n) = b1.mod(n)
        prod.mod(n) = b2.mod(n)
        b1.mod(n) = b2.mod(n)
        b1.congr_mod(b2, n)
    }
}

/// Bridge from is_mod_inv to its underlying congruence.
theorem is_mod_inv_iff_congr(a: Nat, b: Nat, n: Nat) {
    a.is_mod_inv(b, n) = (a * b).congr_mod(Nat.1, n)
}

/// Modular inverses are unique modulo n: any two inverses of the same element
/// are congruent.
theorem mod_inv_unique(a: Nat, n: Nat, b1: Nat, b2: Nat) {
    a.is_mod_inv(b1, n) and a.is_mod_inv(b2, n) implies b1.congr_mod(b2, n)
} by {
    is_mod_inv_iff_congr(a, b1, n)
    is_mod_inv_iff_congr(a, b2, n)
    if a.is_mod_inv(b1, n) and a.is_mod_inv(b2, n) {
        (a * b1).congr_mod(Nat.1, n)
        (a * b2).congr_mod(Nat.1, n)
        mod_inv_unique_congr(a, n, b1, b2)
    }
}

/// The modular inverse of a modulo n: a witnessing Nat with `a * mod_inv(a, n)`
/// congruent to one modulo n when a is coprime to n. When the inverse does not
/// exist (a not coprime to n), the value is the placeholder zero.
let mod_inv(a: Nat, n: Nat) -> b: Nat satisfy {
    if a.coprime(n) {
        a.is_mod_inv(b, n)
    } else {
        b = Nat.0
    }
} by {
    if a.coprime(n) {
        nat_modular_inverse_exists(a, n)
        let b0: Nat satisfy { (a * b0).congr_mod(Nat.1, n) }
        a.is_mod_inv(b0, n)
    } else {
        Nat.0 = Nat.0
    }
}

/// The defined modular inverse satisfies the inverse property when a and n
/// are coprime.
theorem mod_inv_correct(a: Nat, n: Nat) {
    a.coprime(n) implies a.is_mod_inv(mod_inv(a, n), n)
}

/// The defined modular inverse, when interpreted as a product, is congruent
/// to one modulo n for coprime arguments.
theorem mod_inv_mul_congr_one(a: Nat, n: Nat) {
    a.coprime(n) implies (a * mod_inv(a, n)).congr_mod(Nat.1, n)
}

/// Helper: multiplying both sides of `a * x ≡ a * y` on the left by a fresh
/// factor `c` gives `c * (a * x) ≡ c * (a * y)` (mod n).
theorem mul_left_compat(a: Nat, x: Nat, y: Nat, n: Nat, c: Nat) {
    (a * x).congr_mod(a * y, n)
        implies (c * (a * x)).congr_mod(c * (a * y), n)
} by {
    if (a * x).congr_mod(a * y, n) {
        congr_mod_refl(c, n)
        c.congr_mod(c, n)
        congr_mod_mul(c, a * x, c, a * y, n)
    }
}

/// Helper: when `c ≡ 1 (mod n)`, `c * x ≡ 1 * x = x (mod n)`.
theorem mul_one_congr_left(c: Nat, n: Nat, x: Nat) {
    c.congr_mod(Nat.1, n) implies (c * x).congr_mod(x, n)
} by {
    if c.congr_mod(Nat.1, n) {
        congr_mod_refl(x, n)
        x.congr_mod(x, n)
        congr_mod_mul(c, x, Nat.1, x, n)
        (c * x).congr_mod(Nat.1 * x, n)
        Nat.1 * x = x
        (c * x).congr_mod(x, n)
    }
}

/// Helper: from `(b * a) ≡ 1 (mod n)`, both `(b * a) * x ≡ x` and
/// `(b * a) * y ≡ y` hold modulo `n`, and so `x ≡ (b * a) * x` and
/// `(b * a) * y ≡ y`.
theorem cancel_coprime_pin(b: Nat, a: Nat, n: Nat, x: Nat, y: Nat) {
    (b * a).congr_mod(Nat.1, n)
        implies x.congr_mod((b * a) * x, n) and ((b * a) * y).congr_mod(y, n)
} by {
    if (b * a).congr_mod(Nat.1, n) {
        mul_one_congr_left(b * a, n, x)
        ((b * a) * x).congr_mod(x, n)
        congr_mod_symm((b * a) * x, x, n)
        x.congr_mod((b * a) * x, n)
        mul_one_congr_left(b * a, n, y)
        ((b * a) * y).congr_mod(y, n)
    }
}

/// First step of cancel_one_chain: from `c ≡ 1` and `c*x ≡ c*y`, deduce
/// `x ≡ c * y` (mod n).
theorem cancel_one_chain_left(c: Nat, n: Nat, x: Nat, y: Nat) {
    c.congr_mod(Nat.1, n) and (c * x).congr_mod(c * y, n)
        implies x.congr_mod(c * y, n)
} by {
    if c.congr_mod(Nat.1, n) and (c * x).congr_mod(c * y, n) {
        mul_one_congr_left(c, n, x)
        (c * x).congr_mod(x, n)
        congr_mod_symm(c * x, x, n)
        x.congr_mod(c * x, n)
        congr_mod_trans(x, c * x, c * y, n)
    }
}

/// Second step of cancel_one_chain: from `x ≡ c * y` and `c ≡ 1`, deduce
/// `x ≡ y` (mod n).
theorem cancel_one_chain_right(c: Nat, n: Nat, x: Nat, y: Nat) {
    c.congr_mod(Nat.1, n) and x.congr_mod(c * y, n)
        implies x.congr_mod(y, n)
} by {
    if c.congr_mod(Nat.1, n) and x.congr_mod(c * y, n) {
        mul_one_congr_left(c, n, y)
        (c * y).congr_mod(y, n)
        congr_mod_trans(x, c * y, y, n)
    }
}

/// Cancellation core: if `c ≡ 1 (mod n)` and `c * x ≡ c * y (mod n)`, then
/// `x ≡ y (mod n)`.
theorem cancel_one_chain(c: Nat, n: Nat, x: Nat, y: Nat) {
    c.congr_mod(Nat.1, n) and (c * x).congr_mod(c * y, n)
        implies x.congr_mod(y, n)
} by {
    if c.congr_mod(Nat.1, n) and (c * x).congr_mod(c * y, n) {
        cancel_one_chain_left(c, n, x, y)
        x.congr_mod(c * y, n)
        cancel_one_chain_right(c, n, x, y)
    }
}

/// Helper: from `a * x ≡ a * y` and any `b`, derive `(b * a) * x ≡ (b * a) * y`
/// (mod n) by multiplying on the left and re-associating.
theorem cancel_with_inverse_setup(a: Nat, b: Nat, n: Nat, x: Nat, y: Nat) {
    (a * x).congr_mod(a * y, n)
        implies ((b * a) * x).congr_mod((b * a) * y, n)
} by {
    if (a * x).congr_mod(a * y, n) {
        mul_left_compat(a, x, y, n, b)
        (b * (a * x)).congr_mod(b * (a * y), n)
        b * (a * x) = (b * a) * x
        b * (a * y) = (b * a) * y
        ((b * a) * x).congr_mod((b * a) * y, n)
    }
}

/// Cancellation by an explicit inverse: if `b * a ≡ 1 (mod n)` and
/// `a * x ≡ a * y (mod n)`, then `x ≡ y (mod n)`.
theorem cancel_with_inverse(a: Nat, b: Nat, n: Nat, x: Nat, y: Nat) {
    (b * a).congr_mod(Nat.1, n) and (a * x).congr_mod(a * y, n)
        implies x.congr_mod(y, n)
} by {
    if (b * a).congr_mod(Nat.1, n) and (a * x).congr_mod(a * y, n) {
        cancel_with_inverse_setup(a, b, n, x, y)
        ((b * a) * x).congr_mod((b * a) * y, n)
        cancel_one_chain(b * a, n, x, y)
    }
}

/// Modular cancellation: if `a` is coprime to `n`, then `a * x ≡ a * y (mod n)`
/// implies `x ≡ y (mod n)`. The standard proof multiplies both sides by an
/// inverse of `a` modulo `n` and collapses the resulting `1 * x` and `1 * y`.
theorem cancel_coprime(a: Nat, n: Nat, x: Nat, y: Nat) {
    a.coprime(n) and (a * x).congr_mod(a * y, n)
        implies x.congr_mod(y, n)
} by {
    if a.coprime(n) and (a * x).congr_mod(a * y, n) {
        nat_modular_inverse_exists(a, n)
        let b: Nat satisfy { (a * b).congr_mod(Nat.1, n) }
        (a * b).congr_mod(Nat.1, n)
        b * a = a * b
        (b * a).congr_mod(Nat.1, n)
        (a * x).congr_mod(a * y, n)
        cancel_with_inverse(a, b, n, x, y)
        x.congr_mod(y, n)
    }
}

/// Helper: from `a * b ≡ 1 (mod n)`, derive `(a * b).coprime(n)` via the
/// invariance of coprimality under modular reduction.
theorem inverse_imp_product_coprime(a: Nat, b: Nat, n: Nat) {
    (a * b).congr_mod(Nat.1, n) implies (a * b).coprime(n)
} by {
    if (a * b).congr_mod(Nat.1, n) {
        (a * b).mod(n) = Nat.1.mod(n)
        coprime_one_left(n)
        Nat.1.coprime(n)
        coprime_mod_iff(Nat.1, n)
        (Nat.1.mod(n)).coprime(n)
        (a * b).mod(n).coprime(n)
        coprime_mod_iff(a * b, n)
        (a * b).coprime(n)
    }
}

/// Helper: split (a * b).coprime(n) into b.coprime(n) via commutativity and
/// coprime_mul_imp_right.
theorem product_coprime_imp_right(a: Nat, b: Nat, n: Nat) {
    (a * b).coprime(n) implies b.coprime(n)
} by {
    if (a * b).coprime(n) {
        coprime_comm(a * b, n)
        n.coprime(a * b)
        coprime_mul_imp_right(n, a, b)
        n.coprime(b)
        coprime_comm(n, b)
    }
}

/// If `a * b ≡ 1 (mod n)` then `b` is coprime to `n`.
theorem inverse_imp_coprime(a: Nat, b: Nat, n: Nat) {
    (a * b).congr_mod(Nat.1, n) implies b.coprime(n)
} by {
    if (a * b).congr_mod(Nat.1, n) {
        inverse_imp_product_coprime(a, b, n)
        (a * b).coprime(n)
        product_coprime_imp_right(a, b, n)
    }
}

/// `mod_inv(a, n)` is coprime to `n` whenever `a` is.
theorem mod_inv_coprime(a: Nat, n: Nat) {
    a.coprime(n) implies (mod_inv(a, n)).coprime(n)
} by {
    if a.coprime(n) {
        mod_inv_mul_congr_one(a, n)
        (a * mod_inv(a, n)).congr_mod(Nat.1, n)
        inverse_imp_coprime(a, mod_inv(a, n), n)
    }
}

/// Helper: the product of two numbers coprime to `n` is coprime to `n`.
theorem coprime_mul_right_pair(a: Nat, b: Nat, n: Nat) {
    a.coprime(n) and b.coprime(n) implies (a * b).coprime(n)
} by {
    if a.coprime(n) and b.coprime(n) {
        coprime_comm(a, n)
        n.coprime(a)
        coprime_comm(b, n)
        n.coprime(b)
        coprime_mul(n, a, b)
        n.coprime(a * b)
        coprime_comm(n, a * b)
        (a * b).coprime(n)
    }
}

/// The product of two modular inverses gives an explicit inverse witness for a product.
theorem mod_inv_mul_product_congr_one(a: Nat, b: Nat, n: Nat) {
    a.coprime(n) and b.coprime(n) implies
        (a * b * (mod_inv(b, n) * mod_inv(a, n))).congr_mod(Nat.1, n)
} by {
    if a.coprime(n) and b.coprime(n) {
        let ia: Nat = mod_inv(a, n)
        let ib: Nat = mod_inv(b, n)
        mod_inv_mul_congr_one(a, n)
        (a * ia).congr_mod(Nat.1, n)
        mod_inv_mul_congr_one(b, n)
        (b * ib).congr_mod(Nat.1, n)
        congr_mod_mul(a * ia, b * ib, Nat.1, Nat.1, n)
        ((a * ia) * (b * ib)).congr_mod(Nat.1 * Nat.1, n)
        Nat.1 * Nat.1 = Nat.1
        ((a * ia) * (b * ib)).congr_mod(Nat.1, n)
        (a * ia) * (b * ib) = a * (ia * b) * ib
        ia * b = b * ia
        a * (ia * b) * ib = a * (b * ia) * ib
        a * (b * ia) * ib = a * b * ia * ib
        a * b * ia * ib = a * b * (ia * ib)
        ia * ib = ib * ia
        a * b * (ia * ib) = a * b * (ib * ia)
        (a * ia) * (b * ib) = a * b * (ib * ia)
        (a * b * (ib * ia)).congr_mod(Nat.1, n)
        ib = mod_inv(b, n)
        ia = mod_inv(a, n)
        (a * b * (mod_inv(b, n) * mod_inv(a, n))).congr_mod(Nat.1, n)
    }
}

/// The canonical inverse of a product is congruent to the reversed product of inverses.
theorem mod_inv_mul_congr(a: Nat, b: Nat, n: Nat) {
    a.coprime(n) and b.coprime(n) implies
        mod_inv(a * b, n).congr_mod(mod_inv(b, n) * mod_inv(a, n), n)
} by {
    if a.coprime(n) and b.coprime(n) {
        let prod: Nat = a * b
        let inv_prod: Nat = mod_inv(prod, n)
        let candidate: Nat = mod_inv(b, n) * mod_inv(a, n)
        coprime_mul_right_pair(a, b, n)
        prod.coprime(n)
        mod_inv_mul_congr_one(prod, n)
        (prod * inv_prod).congr_mod(Nat.1, n)
        mod_inv_mul_product_congr_one(a, b, n)
        (a * b * (mod_inv(b, n) * mod_inv(a, n))).congr_mod(Nat.1, n)
        prod * candidate = a * b * (mod_inv(b, n) * mod_inv(a, n))
        (prod * candidate).congr_mod(Nat.1, n)
        mod_inv_unique_congr(prod, n, inv_prod, candidate)
        inv_prod.congr_mod(candidate, n)
        mod_inv(a * b, n).congr_mod(mod_inv(b, n) * mod_inv(a, n), n)
    }
}

/// The inverse of one is congruent to one modulo `n`.
theorem mod_inv_one_congr(n: Nat) {
    mod_inv(Nat.1, n).congr_mod(Nat.1, n)
} by {
    coprime_one_left(n)
    Nat.1.coprime(n)
    mod_inv_mul_congr_one(Nat.1, n)
    (Nat.1 * mod_inv(Nat.1, n)).congr_mod(Nat.1, n)
    Nat.1 * mod_inv(Nat.1, n) = mod_inv(Nat.1, n)
    mod_inv(Nat.1, n).congr_mod(Nat.1, n)
}

/// Congruent coprime values have congruent modular inverses.
theorem mod_inv_congr(a: Nat, b: Nat, n: Nat) {
    a.congr_mod(b, n) and a.coprime(n) and b.coprime(n) implies
        mod_inv(a, n).congr_mod(mod_inv(b, n), n)
} by {
    if a.congr_mod(b, n) and a.coprime(n) and b.coprime(n) {
        let ia: Nat = mod_inv(a, n)
        let ib: Nat = mod_inv(b, n)
        congr_mod_refl(ia, n)
        ia.congr_mod(ia, n)
        congr_mod_mul(a, ia, b, ia, n)
        (a * ia).congr_mod(b * ia, n)
        mod_inv_mul_congr_one(a, n)
        (a * ia).congr_mod(Nat.1, n)
        congr_mod_symm(a * ia, b * ia, n)
        (b * ia).congr_mod(a * ia, n)
        congr_mod_trans(b * ia, a * ia, Nat.1, n)
        (b * ia).congr_mod(Nat.1, n)
        mod_inv_mul_congr_one(b, n)
        (b * ib).congr_mod(Nat.1, n)
        mod_inv_unique_congr(b, n, ia, ib)
        ia.congr_mod(ib, n)
        mod_inv(a, n).congr_mod(mod_inv(b, n), n)
    }
}

/// Taking the modular inverse twice returns the original value up to congruence.
theorem mod_inv_involutive_congr(a: Nat, n: Nat) {
    a.coprime(n) implies mod_inv(mod_inv(a, n), n).congr_mod(a, n)
} by {
    if a.coprime(n) {
        let ia: Nat = mod_inv(a, n)
        mod_inv_coprime(a, n)
        ia.coprime(n)
        mod_inv_mul_congr_one(ia, n)
        (ia * mod_inv(ia, n)).congr_mod(Nat.1, n)
        mod_inv_mul_congr_one(a, n)
        (a * ia).congr_mod(Nat.1, n)
        a * ia = ia * a
        (ia * a).congr_mod(Nat.1, n)
        mod_inv_unique_congr(ia, n, mod_inv(ia, n), a)
        mod_inv(ia, n).congr_mod(a, n)
        mod_inv(mod_inv(a, n), n).congr_mod(a, n)
    }
}
