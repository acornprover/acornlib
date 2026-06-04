from nat import Nat, gcd_mul_lcm, gcd_divides_left, gcd_divides_right, gcd_comm,
    gcd_nonzero_left, cofactor, gcd_one_left, gcd_one_right, alt_suc_ne_zero,
    zero_divides
from number_theory.coprime import coprime_divides_of_divides_mul
from nat import divides_symm, divides_trans
numerals Nat

/// Zero is absorbing for lcm on the left.
theorem lcm_zero_left(a: Nat) {
    Nat.0.lcm(a) = Nat.0
} by {
    Nat.0.gcd(a) = a
    if a = Nat.0 {
        Nat.0.gcd(a) = Nat.0
    } else {
        Nat.0.gcd(a) != Nat.0
        Nat.0.gcd(a) * Nat.0.lcm(a) = Nat.0 * a
        Nat.0 * a = Nat.0
        a * Nat.0.lcm(a) = Nat.0
        Nat.0.lcm(a) = Nat.0
    }
}

/// Zero is absorbing for lcm on the right.
theorem lcm_zero_right(a: Nat) {
    a.lcm(Nat.0) = Nat.0
} by {
    a.gcd(Nat.0) = a
    if a = Nat.0 {
        a.gcd(Nat.0) = Nat.0
    } else {
        a.gcd(Nat.0) != Nat.0
        a.gcd(Nat.0) * a.lcm(Nat.0) = a * Nat.0
        a * Nat.0 = Nat.0
        a * a.lcm(Nat.0) = Nat.0
        a.lcm(Nat.0) = Nat.0
    }
}

/// The lcm of two naturals is divisible from the left.
theorem lcm_divides_left(a: Nat, b: Nat) {
    a.divides(a.lcm(b))
} by {
    if a = Nat.0 {
        lcm_zero_left(b)
        a = Nat.0
        Nat.0.lcm(b) = Nat.0
        a.lcm(b) = Nat.0
        Nat.0 * Nat.0 = Nat.0
    } else {
        gcd_nonzero_left(a, b)
        a.gcd(b) != Nat.0
        gcd_divides_right(a, b)
        let k: Nat satisfy { a.gcd(b) * k = b }
        gcd_divides_left(a, b)
        let q: Nat satisfy { a.gcd(b) * q = a }
        a.gcd(b) * a.lcm(b) = a * b
        a * b = q * a.gcd(b) * b
        a.gcd(b) * a.lcm(b) = q * a.gcd(b) * b
        q * a.gcd(b) * b = a.gcd(b) * (q * b)
        a.gcd(b) * a.lcm(b) = a.gcd(b) * (q * b)
        a.lcm(b) = q * b
        b = k * a.gcd(b)
        q * b = q * (k * a.gcd(b))
        q * (k * a.gcd(b)) = (k * q) * a.gcd(b)
        q * a.gcd(b) = a
        (k * q) * a.gcd(b) = k * (q * a.gcd(b))
        k * (q * a.gcd(b)) = k * a
        q * b = k * a
        a.lcm(b) = k * a
        a * k = k * a
        a * k = a.lcm(b)
        a.divides(a.lcm(b))
    }
}

/// The lcm of two naturals is divisible from the right.
theorem lcm_divides_right(a: Nat, b: Nat) {
    b.divides(a.lcm(b))
} by {
    if b = Nat.0 {
        lcm_zero_right(a)
        b * Nat.0 = Nat.0
    } else {
        gcd_comm(a, b)
        gcd_nonzero_left(b, a)
        a.gcd(b) != Nat.0
        gcd_divides_left(a, b)
        let q: Nat satisfy { a.gcd(b) * q = a }
        a.gcd(b) * a.lcm(b) = a * b
        a * b = q * a.gcd(b) * b
        a.gcd(b) * a.lcm(b) = q * a.gcd(b) * b
        q * a.gcd(b) * b = a.gcd(b) * (q * b)
        a.gcd(b) * a.lcm(b) = a.gcd(b) * (q * b)
        a.lcm(b) = q * b
        b * q = q * b
        b * q = a.lcm(b)
        b.divides(a.lcm(b))
    }
}

/// The least common multiple is symmetric in its arguments.
theorem lcm_comm(a: Nat, b: Nat) {
    a.lcm(b) = b.lcm(a)
} by {
    gcd_comm(a, b)
    a.gcd(b) = b.gcd(a)
    gcd_mul_lcm(a, b)
    gcd_mul_lcm(b, a)
    a.gcd(b) * a.lcm(b) = a * b
    b.gcd(a) * b.lcm(a) = b * a
    b * a = a * b
    a.gcd(b) * b.lcm(a) = a * b
    a.gcd(b) * a.lcm(b) = a.gcd(b) * b.lcm(a)
    if a.gcd(b) = Nat.0 {
        gcd_nonzero_left(a, b)
        a = Nat.0
        gcd_comm(a, b)
        gcd_nonzero_left(b, a)
        b = Nat.0
        lcm_zero_left(b)
        lcm_zero_right(b)
        a.lcm(b) = Nat.0
        b.lcm(a) = Nat.0
    } else {
        a.lcm(b) = b.lcm(a)
    }
}

/// One on the left is the identity for lcm.
theorem lcm_one_left(a: Nat) {
    Nat.1.lcm(a) = a
} by {
    gcd_one_left(a)
    Nat.1.gcd(a) = Nat.1
    if a = Nat.0 {
        lcm_zero_right(Nat.1)
        Nat.1.lcm(Nat.0) = Nat.0
    } else {
        Nat.1.gcd(a) != Nat.0
        Nat.1.gcd(a) * Nat.1.lcm(a) = Nat.1 * a
        Nat.1 * Nat.1.lcm(a) = Nat.1 * a
        Nat.1 * a = a
        Nat.1.lcm(a) = a
    }
}

/// One on the right is the identity for lcm.
theorem lcm_one_right(a: Nat) {
    a.lcm(Nat.1) = a
} by {
    gcd_one_right(a)
    a.gcd(Nat.1) = Nat.1
    if a = Nat.0 {
        lcm_zero_left(Nat.1)
        Nat.0.lcm(Nat.1) = Nat.0
    } else {
        a.gcd(Nat.1) != Nat.0
        a.gcd(Nat.1) * a.lcm(Nat.1) = a * Nat.1
        Nat.1 * a.lcm(Nat.1) = a * Nat.1
        a * Nat.1 = a
        a.lcm(Nat.1) = a
    }
}

/// Universal property: any common multiple of a and b is divisible by lcm(a, b).
theorem lcm_divides_of_common(a: Nat, b: Nat, c: Nat) {
    a.divides(c) and b.divides(c) implies a.lcm(b).divides(c)
} by {
    if a.divides(c) and b.divides(c) {
        if a = Nat.0 {
            let z: Nat satisfy { a * z = c }
            c = Nat.0
            lcm_zero_left(b)
            a.lcm(b) = Nat.0
            Nat.0 * Nat.0 = Nat.0
        } else {
        if b = Nat.0 {
            let z: Nat satisfy { b * z = c }
            c = Nat.0
            lcm_zero_right(a)
            a.lcm(b) = Nat.0
        } else {
            gcd_nonzero_left(a, b)
            a.gcd(b) != Nat.0
            gcd_divides_left(a, b)
            gcd_divides_right(a, b)
            let q: Nat satisfy { a.gcd(b) * q = a }
            let k: Nat satisfy { a.gcd(b) * k = b }
            q * a.gcd(b) = a.gcd(b) * q
            k * a.gcd(b) = a.gcd(b) * k
            q * a.gcd(b) = a
            k * a.gcd(b) = b
            cofactor(a, b, q, k)
            q.gcd(k) = Nat.1
            a.gcd(b) * a.lcm(b) = a * b
            a * b = q * a.gcd(b) * b
            a.gcd(b) * a.lcm(b) = q * a.gcd(b) * b
            q * a.gcd(b) * b = a.gcd(b) * (q * b)
            a.gcd(b) * a.lcm(b) = a.gcd(b) * (q * b)
            a.lcm(b) = q * b
            let x: Nat satisfy { a * x = c }
            let y: Nat satisfy { b * y = c }
            b * y = a * x
            b * y = q * a.gcd(b) * x
            b = k * a.gcd(b)
            k * a.gcd(b) * y = q * a.gcd(b) * x
            k * a.gcd(b) * y = a.gcd(b) * (k * y)
            q * a.gcd(b) * x = a.gcd(b) * (q * x)
            a.gcd(b) * (k * y) = a.gcd(b) * (q * x)
            k * y = q * x
            q.divides(k * y)
            q.coprime(k)
            coprime_divides_of_divides_mul(q, k, y)
            q.divides(y)
            let z: Nat satisfy { q * z = y }
            q * b * z = b * (q * z)
            q * b * z = b * y
            q * b * z = c
            a.lcm(b) * z = c
            a.lcm(b).divides(c)
        }
        }
    }
}

/// The least common multiple is associative.
theorem lcm_assoc(a: Nat, b: Nat, c: Nat) {
    (a.lcm(b)).lcm(c) = a.lcm(b.lcm(c))
} by {
    let lhs = (a.lcm(b)).lcm(c)
    let rhs = a.lcm(b.lcm(c))

    lcm_divides_left(a, b)
    lcm_divides_right(a, b)
    lcm_divides_left(b, c)
    lcm_divides_right(b, c)
    lcm_divides_left(a.lcm(b), c)
    lcm_divides_right(a.lcm(b), c)
    a.lcm(b).divides(lhs)
    c.divides(lhs)
    lcm_divides_left(a, b.lcm(c))
    lcm_divides_right(a, b.lcm(c))
    a.divides(rhs)
    b.lcm(c).divides(rhs)

    divides_trans(a, a.lcm(b), lhs)
    a.divides(lhs)
    divides_trans(b, a.lcm(b), lhs)
    b.divides(lhs)
    lcm_divides_of_common(b, c, lhs)
    b.lcm(c).divides(lhs)
    lcm_divides_of_common(a, b.lcm(c), lhs)
    rhs.divides(lhs)

    divides_trans(b, b.lcm(c), rhs)
    b.divides(rhs)
    divides_trans(c, b.lcm(c), rhs)
    c.divides(rhs)
    lcm_divides_of_common(a, b, rhs)
    a.lcm(b).divides(rhs)
    lcm_divides_of_common(a.lcm(b), c, rhs)
    lhs.divides(rhs)

    divides_symm(lhs, rhs)
}
