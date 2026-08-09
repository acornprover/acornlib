from nat import Nat, divides_lte, divides_self, divides_sub,
    add_one_right, add_one_left, suc_sub_one, lt_suc_right, not_lt_zero,
    lte_imp_not_lt, lt_not_ref, add_cancels_left, add_zero_right, add_imp_sub,
    exp_one, one_exp, lt_imp_lte_suc, lte_and_lt
from number_theory.coprime import nat_divides_one_imp_one
from number_theory.factorisation import no_proper_divisor_imp_prime,
    prime_divisor_is_one_or_self, prime_does_not_divide_one
numerals Nat

/// The successor of a number minus the number is one.
theorem suc_sub_self(a: Nat) {
    a.suc - a = Nat.1
} by {
    add_one_left(a)
    Nat.1 + a = a.suc
    add_imp_sub(Nat.1, a, a.suc)
    a.suc - a = Nat.1
}

/// If `a + c = b` with `c` nonzero, then `a` and `b` differ.
theorem lt_ne(a: Nat, b: Nat, c: Nat) {
    a + c = b and c != Nat.0 implies a != b
} by {
    if a + c = b and c != Nat.0 {
        if a = b {
            a + c = a
            a + Nat.0 = a
            add_cancels_left(a, c, Nat.0)
            c = Nat.0
            false
        }
        a != b
    }
}

/// A divisor of two consecutive numbers divides one, hence equals one.
theorem divides_suc_pair_imp_one(d: Nat, m: Nat) {
    d.divides(m) and d.divides(m.suc) implies d = Nat.1
} by {
    if d.divides(m) and d.divides(m.suc) {
        divides_sub(m.suc, m, d)
        suc_sub_self(m)
        m.suc - m = Nat.1
        d.divides(m.suc - m)
        d.divides(Nat.1)
        nat_divides_one_imp_one(d)
        d = Nat.1
    }
}

/// A larger divisor never divides a smaller positive number.
theorem not_divides_of_lt(d: Nat, m: Nat) {
    Nat.0 < m and m < d implies not d.divides(m)
} by {
    if Nat.0 < m and m < d {
        if d.divides(m) {
            divides_lte(d, m)
            m = Nat.0 or d <= m
            if m = Nat.0 {
                false
            } else {
                d <= m
                lte_imp_not_lt(d, m)
                not (m < d)
                false
            }
        }
        not d.divides(m)
    }
}

/// 1 != 5.
theorem one_ne_five {
    Nat.1 != Nat.5
} by {
    lt_ne(Nat.1, Nat.5, Nat.4)
    Nat.1 + Nat.4 = Nat.5
    Nat.4 != Nat.0
}

/// 3 != 1.
theorem three_ne_one {
    Nat.3 != Nat.1
} by {
    lt_ne(Nat.1, Nat.3, Nat.2)
    Nat.1 + Nat.2 = Nat.3
    Nat.2 != Nat.0
}

/// 4 != 1.
theorem four_ne_one {
    Nat.4 != Nat.1
} by {
    lt_ne(Nat.1, Nat.4, Nat.3)
    Nat.1 + Nat.3 = Nat.4
    Nat.3 != Nat.0
}

/// 1 < 5.
theorem lt_one_five {
    Nat.1 < Nat.5
} by {
    Nat.1 + Nat.4 = Nat.5
    exists(c: Nat) { Nat.1 + c = Nat.5 }
    Nat.1 <= Nat.5
    one_ne_five
    Nat.1 != Nat.5
    Nat.1 < Nat.5
}

/// 3 < 5.
theorem lt_three_five {
    Nat.3 < Nat.5
} by {
    Nat.3 + Nat.2 = Nat.5
    exists(c: Nat) { Nat.3 + c = Nat.5 }
    Nat.3 <= Nat.5
    Nat.3 != Nat.5
    Nat.3 < Nat.5
}

/// 0 < 1.
theorem lt_zero_one {
    Nat.0 < Nat.1
}

/// 0 < 2.
theorem lt_zero_two {
    Nat.0 < Nat.2
}

/// 0 < 3.
theorem lt_zero_three {
    Nat.0 < Nat.3
}

/// Two does not divide five.
theorem not_two_divides_five {
    not Nat.2.divides(Nat.5)
} by {
    if Nat.2.divides(Nat.5) {
        Nat.2 * Nat.2 = Nat.4
        exists(c: Nat) { Nat.2 * c = Nat.4 }
        Nat.2.divides(Nat.4)
        divides_suc_pair_imp_one(Nat.2, Nat.4)
        Nat.2 = Nat.1
        Nat.2 != Nat.1
        false
    }
}

/// Three does not divide five.
theorem not_three_divides_five {
    not Nat.3.divides(Nat.5)
} by {
    if Nat.3.divides(Nat.5) {
        Nat.3 * Nat.2 = Nat.6
        exists(c: Nat) { Nat.3 * c = Nat.6 }
        Nat.3.divides(Nat.6)
        divides_suc_pair_imp_one(Nat.3, Nat.6)
        Nat.3 = Nat.1
        three_ne_one
        Nat.3 != Nat.1
        false
    }
}

/// Four does not divide five.
theorem not_four_divides_five {
    not Nat.4.divides(Nat.5)
} by {
    if Nat.4.divides(Nat.5) {
        divides_self(Nat.4)
        Nat.4.divides(Nat.4)
        divides_suc_pair_imp_one(Nat.4, Nat.4)
        Nat.4 = Nat.1
        four_ne_one
        Nat.4 != Nat.1
        false
    }
}

/// 1 < k and k < 3 forces k = 2.
theorem k_range_min(k: Nat) {
    Nat.1 < k and k < Nat.3 implies k = Nat.2
} by {
    if Nat.1 < k and k < Nat.3 {
        lt_suc_right(k, Nat.2)
        k = Nat.2 or k < Nat.2
        if k < Nat.2 {
            false
        } else {
            k = Nat.2
            k = Nat.2
        }
    }
}

/// 1 < k and k < 4 forces k into {2, 3}.
theorem k_range_two_three(k: Nat) {
    Nat.1 < k and k < Nat.4 implies (k = Nat.2 or k = Nat.3)
} by {
    if Nat.1 < k and k < Nat.4 {
        lt_suc_right(k, Nat.3)
        k = Nat.3 or k < Nat.3
        if k < Nat.3 {
            k_range_min(k)
            k = Nat.2
            k = Nat.2 or k = Nat.3
        } else {
            k = Nat.3
            k = Nat.2 or k = Nat.3
        }
    }
}

/// 1 < k and k < 5 forces k into {2, 3, 4}.
theorem k_range_two_four(k: Nat) {
    Nat.1 < k and k < Nat.5 implies (k = Nat.2 or k = Nat.3 or k = Nat.4)
} by {
    if Nat.1 < k and k < Nat.5 {
        lt_suc_right(k, Nat.4)
        k = Nat.4 or k < Nat.4
        if k < Nat.4 {
            k_range_two_three(k)
            k = Nat.2 or k = Nat.3 or k = Nat.4
        } else {
            k = Nat.4
            k = Nat.2 or k = Nat.3 or k = Nat.4
        }
    }
}

/// No number strictly between 1 and 5 divides 5.
theorem five_no_proper_divisor(k: Nat) {
    Nat.1 < k and k < Nat.5 implies not k.divides(Nat.5)
} by {
    if Nat.1 < k and k < Nat.5 {
        lt_suc_right(k, Nat.4)
        k = Nat.4 or k < Nat.4
        if k < Nat.4 {
            lt_suc_right(k, Nat.3)
            k = Nat.3 or k < Nat.3
            if k < Nat.3 {
                lt_suc_right(k, Nat.2)
                k = Nat.2 or k < Nat.2
                if k < Nat.2 {
                    // 1 < k and k < 2 is impossible.
                    lt_imp_lte_suc(Nat.1, k)
                    Nat.2 <= k
                    lte_and_lt(Nat.2, k, Nat.2)
                    Nat.2 < Nat.2
                    lt_not_ref(Nat.2)
                    false
                } else {
                    k = Nat.2
                    not_two_divides_five
                    not Nat.2.divides(Nat.5)
                    not k.divides(Nat.5)
                }
            } else {
                k = Nat.3
                not_three_divides_five
                not Nat.3.divides(Nat.5)
                not k.divides(Nat.5)
            }
        } else {
            k = Nat.4
            not_four_divides_five
            not Nat.4.divides(Nat.5)
            not k.divides(Nat.5)
        }
    }
}

/// Five is prime.
theorem five_is_prime {
    Nat.5.is_prime
} by {
    lt_one_five
    Nat.1 < Nat.5
    forall(k: Nat) {
        five_no_proper_divisor(k)
    }
    no_proper_divisor_imp_prime(Nat.5)
}

/// 2 != 5.
theorem two_ne_five {
    Nat.2 != Nat.5
} by {
    lt_ne(Nat.2, Nat.5, Nat.3)
    Nat.2 + Nat.3 = Nat.5
    Nat.3 != Nat.0
}

/// 1 != 7.
theorem one_ne_seven {
    Nat.1 != Nat.7
} by {
    lt_ne(Nat.1, Nat.7, Nat.6)
    Nat.1 + Nat.6 = Nat.7
    Nat.6 != Nat.0
}

/// 3 != 7.
theorem three_ne_seven {
    Nat.3 != Nat.7
} by {
    lt_ne(Nat.3, Nat.7, Nat.4)
    Nat.3 + Nat.4 = Nat.7
    Nat.4 != Nat.0
}

/// 6 != 1.
theorem six_ne_one {
    Nat.6 != Nat.1
} by {
    lt_ne(Nat.1, Nat.6, Nat.5)
    Nat.1 + Nat.5 = Nat.6
    Nat.5 != Nat.0
}

/// 5 != 7.
theorem five_ne_seven {
    Nat.5 != Nat.7
} by {
    lt_ne(Nat.5, Nat.7, Nat.2)
    Nat.5 + Nat.2 = Nat.7
    Nat.2 != Nat.0
}



/// 2 < 5.
theorem lt_two_five {
    Nat.2 < Nat.5
} by {
    Nat.2 + Nat.3 = Nat.5
    exists(c: Nat) { Nat.2 + c = Nat.5 }
    Nat.2 <= Nat.5
    two_ne_five
    Nat.2 != Nat.5
    Nat.2 < Nat.5
}

/// 1 < 7.
theorem lt_one_seven {
    Nat.1 < Nat.7
} by {
    Nat.1 + Nat.6 = Nat.7
    exists(c: Nat) { Nat.1 + c = Nat.7 }
    Nat.1 <= Nat.7
    one_ne_seven
    Nat.1 != Nat.7
    Nat.1 < Nat.7
}

/// 3 < 7.
theorem lt_three_seven {
    Nat.3 < Nat.7
} by {
    Nat.3 + Nat.4 = Nat.7
    exists(c: Nat) { Nat.3 + c = Nat.7 }
    Nat.3 <= Nat.7
    three_ne_seven
    Nat.3 != Nat.7
    Nat.3 < Nat.7
}


/// Two does not divide seven.
theorem not_two_divides_seven {
    not Nat.2.divides(Nat.7)
} by {
    if Nat.2.divides(Nat.7) {
        Nat.2 * Nat.3 = Nat.6
        exists(c: Nat) { Nat.2 * c = Nat.6 }
        Nat.2.divides(Nat.6)
        divides_suc_pair_imp_one(Nat.2, Nat.6)
        Nat.2 = Nat.1
        Nat.2 != Nat.1
        false
    }
}

/// Three does not divide seven.
theorem not_three_divides_seven {
    not Nat.3.divides(Nat.7)
} by {
    if Nat.3.divides(Nat.7) {
        Nat.3 * Nat.2 = Nat.6
        exists(c: Nat) { Nat.3 * c = Nat.6 }
        Nat.3.divides(Nat.6)
        divides_suc_pair_imp_one(Nat.3, Nat.6)
        Nat.3 = Nat.1
        three_ne_one
        Nat.3 != Nat.1
        false
    }
}

/// Four does not divide seven.
theorem not_four_divides_seven {
    not Nat.4.divides(Nat.7)
} by {
    if Nat.4.divides(Nat.7) {
        Nat.4 * Nat.2 = Nat.8
        exists(c: Nat) { Nat.4 * c = Nat.8 }
        Nat.4.divides(Nat.8)
        divides_suc_pair_imp_one(Nat.4, Nat.8)
        Nat.4 = Nat.1
        four_ne_one
        Nat.4 != Nat.1
        false
    }
}

/// Six does not divide seven.
theorem not_six_divides_seven {
    not Nat.6.divides(Nat.7)
} by {
    if Nat.6.divides(Nat.7) {
        divides_self(Nat.6)
        Nat.6.divides(Nat.6)
        divides_suc_pair_imp_one(Nat.6, Nat.6)
        Nat.6 = Nat.1
        six_ne_one
        Nat.6 != Nat.1
        false
    }
}

/// Five does not divide seven.
theorem not_five_divides_seven {
    not Nat.5.divides(Nat.7)
} by {
    if Nat.5.divides(Nat.7) {
        divides_self(Nat.5)
        Nat.5.divides(Nat.5)
        divides_sub(Nat.7, Nat.5, Nat.5)
        Nat.2 + Nat.5 = Nat.7
        add_imp_sub(Nat.2, Nat.5, Nat.7)
        Nat.7 - Nat.5 = Nat.2
        Nat.5.divides(Nat.2)
        lt_zero_two
        Nat.0 < Nat.2
        lt_two_five
        Nat.2 < Nat.5
        not_divides_of_lt(Nat.5, Nat.2)
        false
    }
}

/// 1 < k and k < 6 forces k into {2, 3, 4, 5}.
theorem k_range_two_five(k: Nat) {
    Nat.1 < k and k < Nat.6 implies (k = Nat.2 or k = Nat.3 or k = Nat.4 or k = Nat.5)
} by {
    if Nat.1 < k and k < Nat.6 {
        lt_suc_right(k, Nat.5)
        k = Nat.5 or k < Nat.5
        if k < Nat.5 {
            k_range_two_four(k)
            k = Nat.2 or k = Nat.3 or k = Nat.4 or k = Nat.5
        } else {
            k = Nat.5
            k = Nat.2 or k = Nat.3 or k = Nat.4 or k = Nat.5
        }
    }
}

/// 1 < k and k < 7 forces k into {2, 3, 4, 5, 6}.
theorem k_range_two_six(k: Nat) {
    Nat.1 < k and k < Nat.7 implies (k = Nat.2 or k = Nat.3 or k = Nat.4 or k = Nat.5 or k = Nat.6)
} by {
    if Nat.1 < k and k < Nat.7 {
        lt_suc_right(k, Nat.6)
        k = Nat.6 or k < Nat.6
        if k < Nat.6 {
            k_range_two_five(k)
            k = Nat.2 or k = Nat.3 or k = Nat.4 or k = Nat.5 or k = Nat.6
        } else {
            k = Nat.6
            k = Nat.2 or k = Nat.3 or k = Nat.4 or k = Nat.5 or k = Nat.6
        }
    }
}

/// Seven is prime, so no number strictly between 1 and 7 divides 7 (by cases).
theorem seven_not_divides_by_case(k: Nat) {
    Nat.1 < k and k < Nat.7 and (k = Nat.2 or k = Nat.3 or k = Nat.4 or k = Nat.5 or k = Nat.6)
        implies not k.divides(Nat.7)
} by {
    if Nat.1 < k and k < Nat.7 and (k = Nat.2 or k = Nat.3 or k = Nat.4 or k = Nat.5 or k = Nat.6) {
        if k = Nat.2 {
            not_two_divides_seven
            not Nat.2.divides(Nat.7)
            not k.divides(Nat.7)
        } else {
            if k = Nat.3 {
                not_three_divides_seven
                not Nat.3.divides(Nat.7)
                not k.divides(Nat.7)
            } else {
                if k = Nat.4 {
                    not_four_divides_seven
                    not Nat.4.divides(Nat.7)
                    not k.divides(Nat.7)
                } else {
                    if k = Nat.5 {
                        not_five_divides_seven
                        not Nat.5.divides(Nat.7)
                        not k.divides(Nat.7)
                    } else {
                        if k = Nat.6 {
                            not_six_divides_seven
                            not Nat.6.divides(Nat.7)
                            not k.divides(Nat.7)
                        } else {
                            false
                        }
                    }
                }
            }
        }
    }
}

/// No number strictly between 1 and 7 divides 7.
theorem seven_no_proper_divisor(k: Nat) {
    Nat.1 < k and k < Nat.7 implies not k.divides(Nat.7)
} by {
    if Nat.1 < k and k < Nat.7 {
        k_range_two_six(k)
        k = Nat.2 or k = Nat.3 or k = Nat.4 or k = Nat.5 or k = Nat.6
        seven_not_divides_by_case(k)
        not k.divides(Nat.7)
    }
}

/// Seven is prime.
theorem seven_is_prime {
    Nat.7.is_prime
} by {
    lt_one_seven
    Nat.1 < Nat.7
    forall(k: Nat) {
        seven_no_proper_divisor(k)
    }
    no_proper_divisor_imp_prime(Nat.7)
}

/// 2^2 = 4.
theorem two_pow_two {
    Nat.2.pow(Nat.2) = Nat.4
}

/// 2^3 = 8.
theorem two_pow_three {
    Nat.2.pow(Nat.3) = Nat.8
} by {
    two_pow_two
    Nat.2.pow(Nat.2) = Nat.4
    Nat.2.pow(Nat.3) = Nat.2 * Nat.2.pow(Nat.2)
    Nat.2 * Nat.4 = Nat.8
    Nat.2.pow(Nat.3) = Nat.8
}

/// 2^4 = 16.
theorem two_pow_four {
    Nat.2.pow(Nat.4) = Nat.16
} by {
    two_pow_three
    Nat.2.pow(Nat.3) = Nat.8
    Nat.2.pow(Nat.4) = Nat.2 * Nat.2.pow(Nat.3)
    Nat.2 * Nat.8 = Nat.16
    Nat.2.pow(Nat.4) = Nat.16
}

/// 3^2 = 9.
theorem three_pow_two {
    Nat.3.pow(Nat.2) = Nat.9
}

/// 16 - 1 = 15.
theorem sixteen_sub_one {
    Nat.16 - Nat.1 = Nat.15
} by {
    add_one_right(Nat.15)
    Nat.15 + Nat.1 = Nat.16
    Nat.15.suc = Nat.16
    suc_sub_one(Nat.15)
    Nat.15.suc - Nat.1 = Nat.15
    Nat.16 - Nat.1 = Nat.15
}

/// 8 - 1 = 7.
theorem eight_sub_one {
    Nat.8 - Nat.1 = Nat.7
}

/// A prime never divides a different prime.
theorem prime_not_divides_distinct(p: Nat, q: Nat) {
    p.is_prime and q.is_prime and p != q implies not p.divides(q)
} by {
    if p.is_prime and q.is_prime and p != q {
        if p.divides(q) {
            prime_divisor_is_one_or_self(q, p)
            p = Nat.1 or p = q
            if p = Nat.1 {
                // p is prime, so 1 < p, contradicting p = 1.
                false
            } else {
                p = q
                false
            }
        }
    }
}

/// True if `n` is a power of two: some `k` with `n = 2^k`.
define is_power_of_two(n: Nat) -> Bool {
    exists(k: Nat) { n = Nat.2.pow(k) }
}

/// `p` is a primitive prime divisor of `a^n - b^n`: `p` is prime, `p` divides
/// `a^n - b^n`, and `p` divides none of `a^k - b^k` for `1 <= k < n`.  The
/// bound `1 <= k` excludes the empty difference `a^0 - b^0 = 0`, which every
/// prime divides.
define is_primitive_prime_divisor(p: Nat, a: Nat, b: Nat, n: Nat) -> Bool {
    p.is_prime and p.divides(a.pow(n) - b.pow(n)) and
        forall(k: Nat) {
            Nat.0 < k and k < n implies not p.divides(a.pow(k) - b.pow(k))
        }
}

/// The primitive-divisor predicate unfolds to its defining conjunction.
theorem is_primitive_prime_divisor_unfold(p: Nat, a: Nat, b: Nat, n: Nat) {
    is_primitive_prime_divisor(p, a, b, n) =
        (p.is_prime and p.divides(a.pow(n) - b.pow(n)) and
            forall(k: Nat) {
                Nat.0 < k and k < n implies not p.divides(a.pow(k) - b.pow(k))
            })
}

/// Zsigmondy's theorem: for coprime `a > b >= 1` and `n >= 1`, the difference
/// `a^n - b^n` admits a primitive prime divisor, except in the three classical
/// exceptional families:
///   (1) `n = 1` and `a - b = 1`, in which case `a - b = 1` has no prime
///       divisor at all;
///   (2) `n = 2` and `a + b` is a power of two (for example `3^2 - 1 = 8`,
///       whose only prime divisor `2` already divides `3^1 - 1`);
///   (3) `(a, b, n) = (2, 1, 6)`, where `2^6 - 1 = 63` has only the prime
///       divisors `3` and `7`, which divide `2^2 - 1` and `2^3 - 1`.
/// The general proof (Bang's theorem) is a known hard theorem; this file
/// records the statement together with the special cases proved below.
// theorem zsigmondy(a: Nat, b: Nat, n: Nat) {
//     b < a and b.coprime(a) and Nat.1 <= n and
//         not (n = Nat.1 and a - b = Nat.1) and
//         not (n = Nat.2 and is_power_of_two(a + b)) and
//         not (a = Nat.2 and b = Nat.1 and n = Nat.6)
//         implies exists(p: Nat) { is_primitive_prime_divisor(p, a, b, n) }
// }

/// A prime divisor of `3^2 - 2^2 = 5`: the prime `5` divides `5` and divides
/// neither `3^1 - 2^1 = 1` (there are no other positive exponents below 2).
theorem zsigmondy_three_two_two {
    exists(p: Nat) { is_primitive_prime_divisor(p, Nat.3, Nat.2, Nat.2) }
} by {
    five_is_prime
    Nat.5.is_prime
    three_pow_two
    Nat.3.pow(Nat.2) = Nat.9
    two_pow_two
    Nat.2.pow(Nat.2) = Nat.4
    Nat.9 - Nat.4 = Nat.5
    divides_self(Nat.5)
    Nat.5.divides(Nat.5)
    Nat.5.divides(Nat.3.pow(Nat.2) - Nat.2.pow(Nat.2))
    forall(k: Nat) {
        if Nat.0 < k and k < Nat.2 {
            if Nat.5.divides(Nat.3.pow(k) - Nat.2.pow(k)) {
                lt_suc_right(k, Nat.1)
                k = Nat.1 or k < Nat.1
                if k < Nat.1 {
                    lt_suc_right(k, Nat.0)
                    k = Nat.0 or k < Nat.0
                    if k < Nat.0 {
                        not_lt_zero(k)
                        false
                    }
                    k = Nat.0
                    false
                }
                k = Nat.1
                exp_one(Nat.3)
                Nat.3.pow(Nat.1) = Nat.3
                exp_one(Nat.2)
                Nat.2.pow(Nat.1) = Nat.2
                Nat.3 - Nat.2 = Nat.1
                divides_lte(Nat.5, Nat.1)
                Nat.1 = Nat.0 or Nat.5 <= Nat.1
                if Nat.1 = Nat.0 {
                    false
                } else {
                    Nat.5 <= Nat.1
                    lte_imp_not_lt(Nat.5, Nat.1)
                    not (Nat.1 < Nat.5)
                    lt_one_five
                    Nat.1 < Nat.5
                    false
                }
                false
            }
            not Nat.5.divides(Nat.3.pow(k) - Nat.2.pow(k))
        }
    }
    is_primitive_prime_divisor(Nat.5, Nat.3, Nat.2, Nat.2)
    exists(p: Nat) { is_primitive_prime_divisor(p, Nat.3, Nat.2, Nat.2) }
}

/// A prime divisor of `2^4 - 1 = 15`: the prime `5` divides `15` and divides
/// none of `2^1 - 1 = 1`, `2^2 - 1 = 3`, `2^3 - 1 = 7`.
theorem zsigmondy_two_one_four {
    exists(p: Nat) { is_primitive_prime_divisor(p, Nat.2, Nat.1, Nat.4) }
} by {
    five_is_prime
    Nat.5.is_prime
    two_pow_four
    Nat.2.pow(Nat.4) = Nat.16
    one_exp(Nat.4)
    Nat.1.pow(Nat.4) = Nat.1
    sixteen_sub_one
    Nat.16 - Nat.1 = Nat.15
    Nat.5 * Nat.3 = Nat.15
    Nat.5.divides(Nat.15)
    Nat.5.divides(Nat.2.pow(Nat.4) - Nat.1.pow(Nat.4))
    forall(k: Nat) {
        if Nat.0 < k and k < Nat.4 {
            if Nat.5.divides(Nat.2.pow(k) - Nat.1.pow(k)) {
                one_exp(k)
                Nat.1.pow(k) = Nat.1
                if k = Nat.1 {
                    exp_one(Nat.2)
                    Nat.2.pow(Nat.1) = Nat.2
                    Nat.2 - Nat.1 = Nat.1
                    not_divides_of_lt(Nat.5, Nat.1)
                    lt_zero_one
                    Nat.0 < Nat.1
                    lt_one_five
                    Nat.1 < Nat.5
                    not Nat.5.divides(Nat.1)
                    false
                }
                if k = Nat.2 {
                    two_pow_two
                    Nat.2.pow(Nat.2) = Nat.4
                    Nat.4 - Nat.1 = Nat.3
                    not_divides_of_lt(Nat.5, Nat.3)
                    lt_zero_three
                    Nat.0 < Nat.3
                    lt_three_five
                    Nat.3 < Nat.5
                    not Nat.5.divides(Nat.3)
                    false
                }
                if k = Nat.3 {
                    two_pow_three
                    Nat.2.pow(Nat.3) = Nat.8
                    eight_sub_one
                    Nat.8 - Nat.1 = Nat.7
                    seven_is_prime
                    prime_not_divides_distinct(Nat.5, Nat.7)
                    Nat.5 != Nat.7
                    not Nat.5.divides(Nat.7)
                    false
                }
                // 0 < k and k < 4 forces k into {1, 2, 3}.
                lt_suc_right(k, Nat.3)
                k = Nat.3 or k < Nat.3
                if k < Nat.3 {
                    lt_suc_right(k, Nat.2)
                    k = Nat.2 or k < Nat.2
                    if k < Nat.2 {
                        lt_suc_right(k, Nat.1)
                        k = Nat.1 or k < Nat.1
                        if k < Nat.1 {
                            lt_suc_right(k, Nat.0)
                            k = Nat.0 or k < Nat.0
                            if k < Nat.0 {
                                not_lt_zero(k)
                                false
                            }
                            k = Nat.0
                            false
                        }
                        k = Nat.1
                        false
                    }
                    k = Nat.2
                    false
                }
                k = Nat.3
                false
            }
            not Nat.5.divides(Nat.2.pow(k) - Nat.1.pow(k))
        }
    }
    is_primitive_prime_divisor(Nat.5, Nat.2, Nat.1, Nat.4)
    exists(p: Nat) { is_primitive_prime_divisor(p, Nat.2, Nat.1, Nat.4) }
}

/// A prime divisor of `2^3 - 1 = 7`: the prime `7` divides `7` and divides
/// neither `2^1 - 1 = 1` nor `2^2 - 1 = 3`.
theorem zsigmondy_two_one_three {
    exists(p: Nat) { is_primitive_prime_divisor(p, Nat.2, Nat.1, Nat.3) }
} by {
    seven_is_prime
    Nat.7.is_prime
    two_pow_three
    Nat.2.pow(Nat.3) = Nat.8
    one_exp(Nat.3)
    Nat.1.pow(Nat.3) = Nat.1
    eight_sub_one
    Nat.8 - Nat.1 = Nat.7
    divides_self(Nat.7)
    Nat.7.divides(Nat.7)
    Nat.7.divides(Nat.2.pow(Nat.3) - Nat.1.pow(Nat.3))
    forall(k: Nat) {
        if Nat.0 < k and k < Nat.3 {
            if Nat.7.divides(Nat.2.pow(k) - Nat.1.pow(k)) {
                one_exp(k)
                Nat.1.pow(k) = Nat.1
                if k = Nat.1 {
                    exp_one(Nat.2)
                    Nat.2.pow(Nat.1) = Nat.2
                    Nat.2 - Nat.1 = Nat.1
                    not_divides_of_lt(Nat.7, Nat.1)
                    lt_zero_one
                    Nat.0 < Nat.1
                    lt_one_seven
                    Nat.1 < Nat.7
                    not Nat.7.divides(Nat.1)
                    false
                }
                if k = Nat.2 {
                    two_pow_two
                    Nat.2.pow(Nat.2) = Nat.4
                    Nat.4 - Nat.1 = Nat.3
                    not_divides_of_lt(Nat.7, Nat.3)
                    lt_zero_three
                    Nat.0 < Nat.3
                    lt_three_seven
                    Nat.3 < Nat.7
                    not Nat.7.divides(Nat.3)
                    false
                }
                // 0 < k and k < 3 forces k into {1, 2}.
                lt_suc_right(k, Nat.2)
                k = Nat.2 or k < Nat.2
                if k < Nat.2 {
                    lt_suc_right(k, Nat.1)
                    k = Nat.1 or k < Nat.1
                    if k < Nat.1 {
                        lt_suc_right(k, Nat.0)
                        k = Nat.0 or k < Nat.0
                        if k < Nat.0 {
                            not_lt_zero(k)
                            false
                        }
                        k = Nat.0
                        false
                    }
                    k = Nat.1
                    false
                }
                k = Nat.2
                false
            }
            not Nat.7.divides(Nat.2.pow(k) - Nat.1.pow(k))
        }
    }
    is_primitive_prime_divisor(Nat.7, Nat.2, Nat.1, Nat.3)
    exists(p: Nat) { is_primitive_prime_divisor(p, Nat.2, Nat.1, Nat.3) }
}

from nat import divides_trans, add_sub, lte_mul, exp_ne_zero, exp_add,
    has_prime_divisor, gcd_of_prime, small_mod, mod_of_zero, div_imp_mod,
    mul_to_one, lte_antisymm, add_imp_sub_left, exp_zero, suc_ne, add_comm,
    add_assoc, add_to_zero, lte_add_left, one_plus_one, lt_trans
from number_theory.congruence import congr_mod_add, congr_mod_refl, congr_mod_symm,
    congr_mod_trans, congr_mod_zero_of_divides, divides_of_congr_mod_zero,
    congr_mod_zero_iff_divides
from number_theory.multiplicative_order import multiplicative_order_mod,
    multiplicative_order_mod_is_order, multiplicative_order_mod_divides_exponent,
    multiplicative_order_pow_congr_one, is_multiplicative_order_mod
from number_theory.sum_of_two_squares import prime_two_divides_imp_eq_two

/// For `n >= 2`, `2^n` is at least `4`.
theorem two_pow_gte_four(n: Nat) {
    Nat.2 <= n implies Nat.4 <= Nat.2.pow(n)
} by {
    if Nat.2 <= n {
        let m: Nat satisfy { Nat.2 + m = n }
        exp_add(Nat.2, Nat.2, m)
        Nat.2.pow(Nat.2 + m) = Nat.2.pow(Nat.2) * Nat.2.pow(m)
        two_pow_two
        Nat.2.pow(Nat.2) = Nat.4
        Nat.2.pow(n) = Nat.4 * Nat.2.pow(m)
        exp_ne_zero(Nat.2, m)
        Nat.2.pow(m) != Nat.0
        lte_mul(Nat.4, Nat.2.pow(m))
        Nat.4 <= Nat.4 * Nat.2.pow(m)
        Nat.4 <= Nat.2.pow(n)
    }
}

/// For `n >= 2`, `2^n - 1` is greater than `1`.
theorem two_pow_sub_one_gt_one(n: Nat) {
    Nat.1 < n implies Nat.1 < Nat.2.pow(n) - Nat.1
} by {
    if Nat.1 < n {
        lt_imp_lte_suc(Nat.1, n)
        Nat.2 <= n
        two_pow_gte_four(n)
        Nat.4 <= Nat.2.pow(n)
        let c: Nat satisfy { Nat.4 + c = Nat.2.pow(n) }
        Nat.3 + c + Nat.1 = Nat.4 + c
        (Nat.3 + c) + Nat.1 = Nat.2.pow(n)
        add_imp_sub_left(Nat.3 + c, Nat.1, Nat.2.pow(n))
        Nat.2.pow(n) - Nat.1 = Nat.3 + c
        Nat.1 + (Nat.2 + c) = Nat.3 + c
        Nat.1 + (Nat.2 + c) = Nat.2.pow(n) - Nat.1
        exists(e: Nat) { Nat.1 + e = Nat.2.pow(n) - Nat.1 }
        Nat.1 <= Nat.2.pow(n) - Nat.1
        Nat.2 + c != Nat.0
        lt_ne(Nat.1, Nat.2.pow(n) - Nat.1, Nat.2 + c)
        Nat.1 != Nat.2.pow(n) - Nat.1
        Nat.1 < Nat.2.pow(n) - Nat.1
    }
}

/// For `n > 0`, `2` divides `2^n`.
theorem two_divides_two_pow(n: Nat) {
    Nat.0 < n implies Nat.2.divides(Nat.2.pow(n))
} by {
    if Nat.0 < n {
        Nat.1 <= n
        add_sub(n, Nat.1)
        n - Nat.1 + Nat.1 = n
        add_comm(Nat.1, n - Nat.1)
        Nat.1 + (n - Nat.1) = n
        exp_add(Nat.2, Nat.1, n - Nat.1)
        Nat.2.pow(n) = Nat.2.pow(Nat.1) * Nat.2.pow(n - Nat.1)
        exp_one(Nat.2)
        Nat.2.pow(Nat.1) = Nat.2
        Nat.2.pow(n) = Nat.2 * Nat.2.pow(n - Nat.1)
        exists(c: Nat) { Nat.2 * c = Nat.2.pow(n) }
        Nat.2.divides(Nat.2.pow(n))
    }
}

/// One is at most `2^n`.
theorem one_le_two_pow(n: Nat) {
    Nat.1 <= Nat.2.pow(n)
} by {
    exp_ne_zero(Nat.2, n)
    Nat.2.pow(n) != Nat.0
    exists(c: Nat) { Nat.0 + c = Nat.2.pow(n) }
    Nat.0 <= Nat.2.pow(n)
    Nat.0 != Nat.2.pow(n)
    Nat.0 < Nat.2.pow(n)
    lt_imp_lte_suc(Nat.0, Nat.2.pow(n))
    Nat.1 <= Nat.2.pow(n)
}

/// A prime other than two is coprime with two.
theorem two_coprime_prime(p: Nat) {
    p.is_prime and p != Nat.2 implies Nat.2.coprime(p)
} by {
    if p.is_prime and p != Nat.2 {
        gcd_of_prime(Nat.2, p)
        if Nat.2.gcd(p) = Nat.1 {
            Nat.2.coprime(p)
        } else {
            Nat.2.divides(p)
            prime_two_divides_imp_eq_two(p)
            p = Nat.2
            false
        }
    }
}

/// 1 < 2.
theorem lt_one_two {
    Nat.1 < Nat.2
} by {
    one_plus_one
    Nat.1 + Nat.1 = Nat.2
    exists(c: Nat) { Nat.1 + c = Nat.2 }
    Nat.1 <= Nat.2
    Nat.1 != Nat.2
    Nat.1 < Nat.2
}

/// A prime other than two is at least three.
theorem two_lt_prime(p: Nat) {
    p.is_prime and p != Nat.2 implies Nat.2 < p
} by {
    if p.is_prime and p != Nat.2 {
        Nat.1 < p
        lt_imp_lte_suc(Nat.1, p)
        Nat.2 <= p
        Nat.2 != p
        Nat.2 < p
    }
}

/// Two is never congruent to one modulo a prime.
theorem two_not_congr_one_mod_prime(p: Nat) {
    p.is_prime implies not Nat.2.congr_mod(Nat.1, p)
} by {
    if p.is_prime {
        if Nat.2.congr_mod(Nat.1, p) {
            Nat.2.mod(p) = Nat.1.mod(p)
            if p = Nat.2 {
                divides_self(Nat.2)
                Nat.2.divides(Nat.2)
                div_imp_mod(Nat.2, Nat.2)
                Nat.2.mod(Nat.2) = Nat.0
                lt_one_two
                Nat.1 < Nat.2
                small_mod(Nat.1, Nat.2)
                Nat.1.mod(Nat.2) = Nat.1
                false
            } else {
                two_lt_prime(p)
                Nat.2 < p
                small_mod(Nat.2, p)
                Nat.2.mod(p) = Nat.2
                Nat.1 < p
                small_mod(Nat.1, p)
                Nat.1.mod(p) = Nat.1
                Nat.2 = Nat.1
                false
            }
            false
        }
        not Nat.2.congr_mod(Nat.1, p)
    }
}

/// Zsigmondy's theorem for `(a, b) = (2, 1)` and prime `n >= 2`: the
/// Mersenne number `2^n - 1` admits a primitive prime divisor.  Every prime
/// divisor `p` of `2^n - 1` has multiplicative order exactly `n` modulo `p`
/// (the order divides the prime `n` and is not `1`), hence divides none of
/// `2^k - 1` with `k < n`.  The full family `n >= 2` has the single
/// exceptional case `n = 6` from Zsigmondy's theorem.
theorem zsigmondy_two_one_prime(n: Nat) {
    n.is_prime implies exists(p: Nat) { is_primitive_prime_divisor(p, Nat.2, Nat.1, n) }
} by {
    if n.is_prime {
        // 2^n - 1 > 1, so it has a prime divisor p.
        Nat.1 < n
        two_pow_sub_one_gt_one(n)
        Nat.1 < Nat.2.pow(n) - Nat.1
        has_prime_divisor(Nat.2.pow(n) - Nat.1)
        let p: Nat satisfy { p.is_prime and p.divides(Nat.2.pow(n) - Nat.1) }
        p.is_prime
        p.divides(Nat.2.pow(n) - Nat.1)
        // p is not 2: 2 divides 2^n but not 2^n - 1.
        if p = Nat.2 {
            Nat.1 < n
            lt_zero_one
            Nat.0 < Nat.1
            lt_trans(Nat.0, Nat.1, n)
            Nat.0 < n
            two_divides_two_pow(n)
            Nat.2.divides(Nat.2.pow(n))
            divides_sub(Nat.2.pow(n), Nat.2.pow(n) - Nat.1, Nat.2)
            Nat.2.divides(Nat.2.pow(n) - (Nat.2.pow(n) - Nat.1))
            one_le_two_pow(n)
            Nat.1 <= Nat.2.pow(n)
            add_sub(Nat.2.pow(n), Nat.1)
            Nat.2.pow(n) - Nat.1 + Nat.1 = Nat.2.pow(n)
            add_imp_sub_left(Nat.2.pow(n) - Nat.1, Nat.1, Nat.2.pow(n))
            Nat.2.pow(n) - (Nat.2.pow(n) - Nat.1) = Nat.1
            Nat.2.divides(Nat.1)
            nat_divides_one_imp_one(Nat.2)
            Nat.2 = Nat.1
            false
        }
        p != Nat.2
        // 2 is coprime with p.
        two_coprime_prime(p)
        Nat.2.coprime(p)
        // 2^n is congruent to 1 modulo p.
        congr_mod_zero_of_divides(p, Nat.2.pow(n) - Nat.1)
        (Nat.2.pow(n) - Nat.1).congr_mod(Nat.0, p)
        congr_mod_refl(Nat.1, p)
        Nat.1.congr_mod(Nat.1, p)
        congr_mod_add(Nat.2.pow(n) - Nat.1, Nat.1, Nat.0, Nat.1, p)
        (Nat.2.pow(n) - Nat.1 + Nat.1).congr_mod(Nat.0 + Nat.1, p)
        one_le_two_pow(n)
        Nat.1 <= Nat.2.pow(n)
        add_sub(Nat.2.pow(n), Nat.1)
        Nat.2.pow(n) - Nat.1 + Nat.1 = Nat.2.pow(n)
        Nat.0 + Nat.1 = Nat.1
        Nat.2.pow(n).congr_mod(Nat.1, p)
        // The multiplicative order d of 2 modulo p divides n.
        Nat.1 < p
        if p = Nat.0 {
            not_lt_zero(Nat.1)
            false
        }
        p != Nat.0
        multiplicative_order_mod_is_order(Nat.2, p)
        is_multiplicative_order_mod(Nat.2, p, multiplicative_order_mod(Nat.2, p))
        multiplicative_order_mod_divides_exponent(Nat.2, p, n)
        multiplicative_order_mod(Nat.2, p).divides(n)
        // n is prime, so d = 1 or d = n.
        prime_divisor_is_one_or_self(n, multiplicative_order_mod(Nat.2, p))
        multiplicative_order_mod(Nat.2, p) = Nat.1 or multiplicative_order_mod(Nat.2, p) = n
        if multiplicative_order_mod(Nat.2, p) = Nat.1 {
            // d = 1 would make 2 congruent to 1 modulo p.
            multiplicative_order_mod_is_order(Nat.2, p)
            is_multiplicative_order_mod(Nat.2, p, multiplicative_order_mod(Nat.2, p))
            multiplicative_order_pow_congr_one(Nat.2, p, multiplicative_order_mod(Nat.2, p))
            Nat.2.pow(multiplicative_order_mod(Nat.2, p)).congr_mod(Nat.1, p)
            exp_one(Nat.2)
            Nat.2.pow(Nat.1) = Nat.2
            Nat.2.congr_mod(Nat.1, p)
            two_not_congr_one_mod_prime(p)
            false
        }
        multiplicative_order_mod(Nat.2, p) = n
        // Every prime divisor is primitive: for 0 < k < n, p does not
        // divide 2^k - 1.
        forall(k: Nat) {
            if Nat.0 < k and k < n {
                if p.divides(Nat.2.pow(k) - Nat.1.pow(k)) {
                    one_exp(k)
                    Nat.1.pow(k) = Nat.1
                    p.divides(Nat.2.pow(k) - Nat.1)
                    congr_mod_zero_of_divides(p, Nat.2.pow(k) - Nat.1)
                    (Nat.2.pow(k) - Nat.1).congr_mod(Nat.0, p)
                    congr_mod_refl(Nat.1, p)
                    Nat.1.congr_mod(Nat.1, p)
                    congr_mod_add(Nat.2.pow(k) - Nat.1, Nat.1, Nat.0, Nat.1, p)
                    (Nat.2.pow(k) - Nat.1 + Nat.1).congr_mod(Nat.0 + Nat.1, p)
                    one_le_two_pow(k)
                    Nat.1 <= Nat.2.pow(k)
                    add_sub(Nat.2.pow(k), Nat.1)
                    Nat.2.pow(k) - Nat.1 + Nat.1 = Nat.2.pow(k)
                    Nat.0 + Nat.1 = Nat.1
                    Nat.2.pow(k).congr_mod(Nat.1, p)
                    multiplicative_order_mod_divides_exponent(Nat.2, p, k)
                    multiplicative_order_mod(Nat.2, p).divides(k)
                    n.divides(k)
                    divides_lte(n, k)
                    k = Nat.0 or n <= k
                    if k = Nat.0 {
                        false
                    } else {
                        n <= k
                        lte_imp_not_lt(n, k)
                        not (k < n)
                        false
                    }
                    false
                }
                not p.divides(Nat.2.pow(k) - Nat.1.pow(k))
            }
        }
        // Assemble the primitive-divisor predicate.
        is_primitive_prime_divisor_unfold(p, Nat.2, Nat.1, n)
        one_exp(n)
        Nat.1.pow(n) = Nat.1
        p.divides(Nat.2.pow(n) - Nat.1.pow(n))
        is_primitive_prime_divisor(p, Nat.2, Nat.1, n)
        exists(q: Nat) { is_primitive_prime_divisor(q, Nat.2, Nat.1, n) }
    }
}
