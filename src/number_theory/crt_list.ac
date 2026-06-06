from number_theory.congruence import Nat, congr_mod_symm, congr_mod_trans
from nat import divides_self, divides_mul, mul_to_zero, small_mod
from number_theory.congr_int import nat_congr_mod_iff_int_mod_rel
from number_theory.crt import int_mod_rel_descend, nat_crt_two_moduli,
    nat_congr_combine_coprime
from number_theory.pairwise_coprime import pairwise_coprime, coprime_with_all,
    pairwise_coprime_cons_imp, coprime_with_all_imp_coprime_product
from list import List
from list import product
from pair import Pair
from int import Int
from zmod import int_mod_rel
numerals Nat

/// True if c satisfies every congruence requirement in a list of
/// (modulus, residue) pairs, i.e., for each (m, r) pair, c is congruent to
/// r modulo m.
define satisfies_all(c: Nat, system: List[Pair[Nat, Nat]]) -> Bool {
    match system {
        List.nil {
            true
        }
        List.cons(head, tail) {
            c.congr_mod(head.second, head.first) and satisfies_all(c, tail)
        }
    }
}

/// The empty system is satisfied by anything.
theorem satisfies_all_nil(c: Nat) {
    satisfies_all(c, List.nil[Pair[Nat, Nat]])
}

/// Forward direction of the cons unfold for satisfies_all.
theorem satisfies_all_cons_imp(c: Nat, head: Pair[Nat, Nat], tail: List[Pair[Nat, Nat]]) {
    satisfies_all(c, List.cons(head, tail)) implies
        c.congr_mod(head.second, head.first) and satisfies_all(c, tail)
} by {
    if satisfies_all(c, List.cons(head, tail)) {
        c.congr_mod(head.second, head.first) and satisfies_all(c, tail)
    }
}

/// Building up satisfies_all from the cons-pieces.
theorem satisfies_all_cons_intro(c: Nat, head: Pair[Nat, Nat], tail: List[Pair[Nat, Nat]]) {
    c.congr_mod(head.second, head.first) and satisfies_all(c, tail) implies
        satisfies_all(c, List.cons(head, tail))
} by {
    if c.congr_mod(head.second, head.first) and satisfies_all(c, tail) {
        satisfies_all(c, List.cons(head, tail))
    }
}

/// CRT for the empty system: every Nat trivially solves the empty system.
theorem nat_crt_list_nil {
    exists(c: Nat) { satisfies_all(c, List.nil[Pair[Nat, Nat]]) }
} by {
    satisfies_all_nil(Nat.0)
}

/// Project a system of (modulus, residue) pairs to its list of moduli.
define system_moduli(system: List[Pair[Nat, Nat]]) -> List[Nat] {
    match system {
        List.nil {
            List.nil[Nat]
        }
        List.cons(head, tail) {
            List.cons(head.first, system_moduli(tail))
        }
    }
}

/// The combined modulus for a system: the product of all moduli.
define system_modulus(system: List[Pair[Nat, Nat]]) -> Nat {
    product[Nat](system_moduli(system))
}

/// The empty system has moduli list nil and combined modulus 1.
theorem system_modulus_nil {
    system_modulus(List.nil[Pair[Nat, Nat]]) = Nat.1
}

/// The combined modulus of a cons system factors as head modulus times
/// the combined modulus of the tail.
theorem system_modulus_cons(head: Pair[Nat, Nat], tail: List[Pair[Nat, Nat]]) {
    system_modulus(List.cons(head, tail)) = head.first * system_modulus(tail)
} by {
    system_moduli(List.cons(head, tail)) = List.cons(head.first, system_moduli(tail))
    product[Nat](List.cons(head.first, system_moduli(tail))) =
        head.first * product[Nat](system_moduli(tail))
}

/// Descending a Nat congruence along divisibility of moduli.
theorem nat_congr_mod_descend(m: Nat, k: Nat, x: Nat, y: Nat) {
    m.divides(k) and x.congr_mod(y, k) implies x.congr_mod(y, m)
} by {
    if m.divides(k) and x.congr_mod(y, k) {
        nat_congr_mod_iff_int_mod_rel(x, y, k)
        int_mod_rel(k, Int.from_nat(x), Int.from_nat(y))
        int_mod_rel_descend(m, k, Int.from_nat(x), Int.from_nat(y))
        int_mod_rel(m, Int.from_nat(x), Int.from_nat(y))
        nat_congr_mod_iff_int_mod_rel(x, y, m)
        x.congr_mod(y, m)
    }
}

/// Split a system_modulus(cons) congruence into the head-modulus and
/// tail-modulus pieces.
theorem cons_modulus_split(c: Nat, c0: Nat, head: Pair[Nat, Nat], tail: List[Pair[Nat, Nat]]) {
    c.congr_mod(c0, system_modulus(List.cons(head, tail)))
        implies c.congr_mod(c0, head.first) and c.congr_mod(c0, system_modulus(tail))
} by {
    if c.congr_mod(c0, system_modulus(List.cons(head, tail))) {
        system_modulus_cons(head, tail)
        let big_mod: Nat = head.first * system_modulus(tail)
        c.congr_mod(c0, big_mod)
        divides_self(head.first)
        divides_mul(head.first, system_modulus(tail), head.first)
        head.first.divides(big_mod)
        nat_congr_mod_descend(head.first, big_mod, c, c0)
        divides_self(system_modulus(tail))
        divides_mul(system_modulus(tail), head.first, system_modulus(tail))
        system_modulus(tail) * head.first = head.first * system_modulus(tail)
        system_modulus(tail).divides(big_mod)
        nat_congr_mod_descend(system_modulus(tail), big_mod, c, c0)
        c.congr_mod(c0, head.first) and c.congr_mod(c0, system_modulus(tail))
    }
}

/// If c is congruent to c0 modulo the combined system modulus, and c0
/// satisfies the system, then c also satisfies the system.
theorem satisfies_all_descend(c: Nat, c0: Nat, system: List[Pair[Nat, Nat]]) {
    c.congr_mod(c0, system_modulus(system)) and satisfies_all(c0, system)
        implies satisfies_all(c, system)
} by {
    define p(s: List[Pair[Nat, Nat]]) -> Bool {
        c.congr_mod(c0, system_modulus(s)) and satisfies_all(c0, s)
            implies satisfies_all(c, s)
    }
    satisfies_all_nil(c)
    p(List.nil[Pair[Nat, Nat]])
    forall(head: Pair[Nat, Nat], tail: List[Pair[Nat, Nat]]) {
        if p(tail) {
            if c.congr_mod(c0, system_modulus(List.cons(head, tail)))
                and satisfies_all(c0, List.cons(head, tail)) {
                cons_modulus_split(c, c0, head, tail)
                c.congr_mod(c0, head.first)
                c.congr_mod(c0, system_modulus(tail))
                satisfies_all_cons_imp(c0, head, tail)
                c0.congr_mod(head.second, head.first)
                satisfies_all(c0, tail)
                satisfies_all(c, tail)
                congr_mod_trans(c, c0, head.second, head.first)
                c.congr_mod(head.second, head.first)
                satisfies_all_cons_intro(c, head, tail)
                satisfies_all(c, List.cons(head, tail))
            }
            p(List.cons(head, tail))
        }
    }
    List.induction(function(l: List[Pair[Nat, Nat]]) { p(l) })
    forall(l: List[Pair[Nat, Nat]]) {
        p(l)
    }
    p(system)
}

/// Cons-step glue: given the tail's descent already discharged, build
/// the cons descent.
theorem satisfies_all_descend_cons_step(
    c: Nat, c0: Nat, head: Pair[Nat, Nat], tail: List[Pair[Nat, Nat]]
) {
    satisfies_all(c, tail)
        and c.congr_mod(c0, system_modulus(List.cons(head, tail)))
        and satisfies_all(c0, List.cons(head, tail))
        implies satisfies_all(c, List.cons(head, tail))
} by {
    if satisfies_all(c, tail)
        and c.congr_mod(c0, system_modulus(List.cons(head, tail)))
        and satisfies_all(c0, List.cons(head, tail)) {
        cons_modulus_split(c, c0, head, tail)
        c.congr_mod(c0, head.first)
        satisfies_all_cons_imp(c0, head, tail)
        c0.congr_mod(head.second, head.first)
        congr_mod_trans(c, c0, head.second, head.first)
        c.congr_mod(head.second, head.first)
        satisfies_all_cons_intro(c, head, tail)
        satisfies_all(c, List.cons(head, tail))
    }
}



/// True if every modulus in the system is positive (nonzero).
define every_modulus_positive(system: List[Pair[Nat, Nat]]) -> Bool {
    match system {
        List.nil {
            true
        }
        List.cons(head, tail) {
            head.first != Nat.0 and every_modulus_positive(tail)
        }
    }
}

/// Cons unfold for every_modulus_positive.
theorem every_modulus_positive_cons_imp(head: Pair[Nat, Nat], tail: List[Pair[Nat, Nat]]) {
    every_modulus_positive(List.cons(head, tail)) implies
        head.first != Nat.0 and every_modulus_positive(tail)
} by {
    if every_modulus_positive(List.cons(head, tail)) {
        head.first != Nat.0 and every_modulus_positive(tail)
    }
}

/// If every modulus is positive then the combined system modulus is positive.
theorem every_modulus_positive_imp_modulus_nonzero(system: List[Pair[Nat, Nat]]) {
    every_modulus_positive(system) implies system_modulus(system) != Nat.0
} by {
    define p(s: List[Pair[Nat, Nat]]) -> Bool {
        every_modulus_positive(s) implies system_modulus(s) != Nat.0
    }
    system_modulus_nil
    system_modulus(List.nil[Pair[Nat, Nat]]) = Nat.1
    Nat.1 != Nat.0
    p(List.nil[Pair[Nat, Nat]])
    forall(head: Pair[Nat, Nat], tail: List[Pair[Nat, Nat]]) {
        if p(tail) {
            if every_modulus_positive(List.cons(head, tail)) {
                every_modulus_positive_cons_imp(head, tail)
                head.first != Nat.0
                every_modulus_positive(tail)
                system_modulus(tail) != Nat.0
                system_modulus_cons(head, tail)
                system_modulus(List.cons(head, tail)) = head.first * system_modulus(tail)
                mul_to_zero(head.first, system_modulus(tail))
                head.first * system_modulus(tail) != Nat.0
                system_modulus(List.cons(head, tail)) != Nat.0
            }
            p(List.cons(head, tail))
        }
    }
    List.induction(function(l: List[Pair[Nat, Nat]]) { p(l) })
    forall(l: List[Pair[Nat, Nat]]) {
        p(l)
    }
    p(system)
}

/// CRT cons step: from a tail-system solution and the cons hypotheses, build
/// a cons-system solution.
theorem nat_crt_list_cons(head: Pair[Nat, Nat], tail: List[Pair[Nat, Nat]]) {
    pairwise_coprime(system_moduli(List.cons(head, tail)))
        and every_modulus_positive(List.cons(head, tail))
        and (exists(c0: Nat) { satisfies_all(c0, tail) })
        implies exists(c: Nat) { satisfies_all(c, List.cons(head, tail)) }
} by {
    if pairwise_coprime(system_moduli(List.cons(head, tail)))
        and every_modulus_positive(List.cons(head, tail))
        and (exists(c0: Nat) { satisfies_all(c0, tail) }) {
        // Unfold the system_moduli and pairwise_coprime hypotheses.
        system_moduli(List.cons(head, tail)) =
            List.cons(head.first, system_moduli(tail))
        pairwise_coprime_cons_imp(head.first, system_moduli(tail))
        coprime_with_all(head.first, system_moduli(tail))
        coprime_with_all_imp_coprime_product(head.first, system_moduli(tail))
        head.first.coprime(product[Nat](system_moduli(tail)))
        head.first.coprime(system_modulus(tail))
        // Positivity facts.
        every_modulus_positive_cons_imp(head, tail)
        head.first != Nat.0
        every_modulus_positive(tail)
        every_modulus_positive_imp_modulus_nonzero(tail)
        system_modulus(tail) != Nat.0
        // Pull a tail solution.
        let c0: Nat satisfy { satisfies_all(c0, tail) }
        // Apply two-modulus CRT to (head.first, system_modulus(tail)) with
        // residues (head.second, c0).
        nat_crt_two_moduli(head.first, system_modulus(tail), head.second, c0)
        let c: Nat satisfy {
            c.congr_mod(head.second, head.first)
                and c.congr_mod(c0, system_modulus(tail))
        }
        c.congr_mod(c0, system_modulus(tail))
        // Descend the system_modulus(tail) congruence into tail satisfaction.
        satisfies_all_descend(c, c0, tail)
        satisfies_all(c, tail)
        // Build the cons satisfaction.
        c.congr_mod(head.second, head.first)
        satisfies_all_cons_intro(c, head, tail)
        satisfies_all(c, List.cons(head, tail))
    }
}

/// List-indexed Chinese remainder theorem on the naturals: a system of
/// pairwise-coprime positive moduli paired with residues has a simultaneous
/// Nat solution.
theorem nat_crt_list(system: List[Pair[Nat, Nat]]) {
    pairwise_coprime(system_moduli(system))
        and every_modulus_positive(system)
        implies exists(c: Nat) { satisfies_all(c, system) }
} by {
    define p(s: List[Pair[Nat, Nat]]) -> Bool {
        pairwise_coprime(system_moduli(s))
            and every_modulus_positive(s)
            implies exists(c: Nat) { satisfies_all(c, s) }
    }
    nat_crt_list_nil
    p(List.nil[Pair[Nat, Nat]])
    forall(head: Pair[Nat, Nat], tail: List[Pair[Nat, Nat]]) {
        if p(tail) {
            if pairwise_coprime(system_moduli(List.cons(head, tail)))
                and every_modulus_positive(List.cons(head, tail)) {
                // Tail hypotheses follow.
                system_moduli(List.cons(head, tail)) =
                    List.cons(head.first, system_moduli(tail))
                pairwise_coprime_cons_imp(head.first, system_moduli(tail))
                pairwise_coprime(system_moduli(tail))
                every_modulus_positive_cons_imp(head, tail)
                every_modulus_positive(tail)
                // IH gives a tail solution.
                exists(c0: Nat) { satisfies_all(c0, tail) }
                nat_crt_list_cons(head, tail)
                exists(c: Nat) { satisfies_all(c, List.cons(head, tail)) }
            }
            p(List.cons(head, tail))
        }
    }
    List.induction(function(l: List[Pair[Nat, Nat]]) { p(l) })
    forall(l: List[Pair[Nat, Nat]]) {
        p(l)
    }
    p(system)
}

/// Every two natural numbers are congruent modulo one.
theorem nat_congr_mod_one(a: Nat, b: Nat) {
    a.congr_mod(b, Nat.1)
} by {
    let diff: Int = Int.from_nat(a) - Int.from_nat(b)
    diff * Int.1 = diff
    Int.1.divides(diff)
    Int.from_nat(Nat.1) = Int.1
    int_mod_rel(Nat.1, Int.from_nat(a), Int.from_nat(b))
    nat_congr_mod_iff_int_mod_rel(a, b, Nat.1)
    a.congr_mod(b, Nat.1)
}

/// In a pairwise-coprime cons system, the head modulus is coprime to the
/// product of the tail moduli.
theorem cons_head_coprime_tail_system_modulus(head: Pair[Nat, Nat], tail: List[Pair[Nat, Nat]]) {
    pairwise_coprime(system_moduli(List.cons(head, tail))) implies
        head.first.coprime(system_modulus(tail))
} by {
    if pairwise_coprime(system_moduli(List.cons(head, tail))) {
        system_moduli(List.cons(head, tail)) =
            List.cons(head.first, system_moduli(tail))
        pairwise_coprime_cons_imp(head.first, system_moduli(tail))
        coprime_with_all(head.first, system_moduli(tail))
        coprime_with_all_imp_coprime_product(head.first, system_moduli(tail))
        head.first.coprime(product[Nat](system_moduli(tail)))
        head.first.coprime(system_modulus(tail))
    }
}

/// Two solutions of a cons system are congruent modulo the head modulus.
theorem cons_solutions_congruent_mod_head(
    head: Pair[Nat, Nat], tail: List[Pair[Nat, Nat]], c1: Nat, c2: Nat
) {
    satisfies_all(c1, List.cons(head, tail))
        and satisfies_all(c2, List.cons(head, tail))
        implies c1.congr_mod(c2, head.first)
} by {
    if satisfies_all(c1, List.cons(head, tail))
        and satisfies_all(c2, List.cons(head, tail)) {
        satisfies_all_cons_imp(c1, head, tail)
        satisfies_all_cons_imp(c2, head, tail)
        c1.congr_mod(head.second, head.first)
        c2.congr_mod(head.second, head.first)
        congr_mod_symm(c2, head.second, head.first)
        head.second.congr_mod(c2, head.first)
        congr_mod_trans(c1, head.second, c2, head.first)
        c1.congr_mod(c2, head.first)
    }
}

/// Congruences modulo the head and tail product combine to the cons-system
/// product modulus when the two factors are coprime.
theorem combine_head_tail_congruence_mod_system_modulus(
    head: Pair[Nat, Nat], tail: List[Pair[Nat, Nat]], c1: Nat, c2: Nat
) {
    head.first.coprime(system_modulus(tail))
        and c1.congr_mod(c2, head.first)
        and c1.congr_mod(c2, system_modulus(tail))
        implies c1.congr_mod(c2, system_modulus(List.cons(head, tail)))
} by {
    if head.first.coprime(system_modulus(tail))
        and c1.congr_mod(c2, head.first)
        and c1.congr_mod(c2, system_modulus(tail)) {
        nat_congr_combine_coprime(head.first, system_modulus(tail), c1, c2)
        c1.congr_mod(c2, head.first * system_modulus(tail))
        system_modulus_cons(head, tail)
        c1.congr_mod(c2, system_modulus(List.cons(head, tail)))
    }
}

/// Any two solutions of a pairwise-coprime positive list system are congruent
/// modulo the combined system modulus.
theorem satisfies_all_unique_mod_system_modulus(system: List[Pair[Nat, Nat]], c1: Nat, c2: Nat) {
    pairwise_coprime(system_moduli(system))
        and every_modulus_positive(system)
        and satisfies_all(c1, system)
        and satisfies_all(c2, system)
        implies c1.congr_mod(c2, system_modulus(system))
} by {
    define p(s: List[Pair[Nat, Nat]]) -> Bool {
        pairwise_coprime(system_moduli(s))
            and every_modulus_positive(s)
            and satisfies_all(c1, s)
            and satisfies_all(c2, s)
            implies c1.congr_mod(c2, system_modulus(s))
    }
    system_modulus_nil
    nat_congr_mod_one(c1, c2)
    p(List.nil[Pair[Nat, Nat]])
    forall(head: Pair[Nat, Nat], tail: List[Pair[Nat, Nat]]) {
        if p(tail) {
            if not p(List.cons(head, tail)) {
                p(List.cons(head, tail)) = (pairwise_coprime(system_moduli(List.cons(head, tail)))
                    and every_modulus_positive(List.cons(head, tail))
                    and satisfies_all(c1, List.cons(head, tail))
                    and satisfies_all(c2, List.cons(head, tail))
                    implies c1.congr_mod(c2, system_modulus(List.cons(head, tail))))
                pairwise_coprime(system_moduli(List.cons(head, tail)))
                every_modulus_positive(List.cons(head, tail))
                satisfies_all(c1, List.cons(head, tail))
                satisfies_all(c2, List.cons(head, tail))
                system_moduli(List.cons(head, tail)) =
                    List.cons(head.first, system_moduli(tail))
                pairwise_coprime_cons_imp(head.first, system_moduli(tail))
                pairwise_coprime(system_moduli(tail))
                every_modulus_positive_cons_imp(head, tail)
                every_modulus_positive(tail)
                satisfies_all_cons_imp(c1, head, tail)
                satisfies_all_cons_imp(c2, head, tail)
                satisfies_all(c1, tail)
                satisfies_all(c2, tail)
                p(tail) = (pairwise_coprime(system_moduli(tail))
                    and every_modulus_positive(tail)
                    and satisfies_all(c1, tail)
                    and satisfies_all(c2, tail)
                    implies c1.congr_mod(c2, system_modulus(tail)))
                c1.congr_mod(c2, system_modulus(tail))
                cons_head_coprime_tail_system_modulus(head, tail)
                head.first.coprime(system_modulus(tail))
                cons_solutions_congruent_mod_head(head, tail, c1, c2)
                c1.congr_mod(c2, head.first)
                combine_head_tail_congruence_mod_system_modulus(head, tail, c1, c2)
                c1.congr_mod(c2, system_modulus(List.cons(head, tail)))
                false
            }
            p(List.cons(head, tail))
        }
        p(tail) implies p(List.cons(head, tail))
    }
    match system {
        List.nil {
            p(List.nil[Pair[Nat, Nat]])
            p(system)
        }
        List.cons(head, tail) {
            List.induction(function(l: List[Pair[Nat, Nat]]) { p(l) })
            forall(l: List[Pair[Nat, Nat]]) {
                p(l)
            }
            p(system)
        }
    }
}

/// A congruence modulo the system modulus is equality for representatives
/// strictly below that positive modulus.
theorem congr_mod_below_system_modulus_eq(system: List[Pair[Nat, Nat]], c1: Nat, c2: Nat) {
    every_modulus_positive(system)
        and c1 < system_modulus(system)
        and c2 < system_modulus(system)
        and c1.congr_mod(c2, system_modulus(system))
        implies c1 = c2
} by {
    if every_modulus_positive(system)
        and c1 < system_modulus(system)
        and c2 < system_modulus(system)
        and c1.congr_mod(c2, system_modulus(system)) {
        every_modulus_positive_imp_modulus_nonzero(system)
        let m: Nat = system_modulus(system)
        m != Nat.0
        c1.mod(m) = c2.mod(m)
        small_mod(c1, m)
        small_mod(c2, m)
        c1.mod(m) = c1
        c2.mod(m) = c2
        c1 = c2
    }
}

/// Any two normalized solutions of a pairwise-coprime positive list system are
/// equal.
theorem satisfies_all_unique_below_system_modulus(system: List[Pair[Nat, Nat]], c1: Nat, c2: Nat) {
    pairwise_coprime(system_moduli(system))
        and every_modulus_positive(system)
        and satisfies_all(c1, system)
        and satisfies_all(c2, system)
        and c1 < system_modulus(system)
        and c2 < system_modulus(system)
        implies c1 = c2
} by {
    if pairwise_coprime(system_moduli(system))
        and every_modulus_positive(system)
        and satisfies_all(c1, system)
        and satisfies_all(c2, system)
        and c1 < system_modulus(system)
        and c2 < system_modulus(system) {
        satisfies_all_unique_mod_system_modulus(system, c1, c2)
        c1.congr_mod(c2, system_modulus(system))
        congr_mod_below_system_modulus_eq(system, c1, c2)
        c1 = c2
    }
}

/// Every solution is congruent modulo the combined modulus to some solution
/// obtained from the list CRT existence theorem.
theorem satisfies_all_congruent_to_crt_list_solution(system: List[Pair[Nat, Nat]], c: Nat) {
    pairwise_coprime(system_moduli(system))
        and every_modulus_positive(system)
        and satisfies_all(c, system)
        implies exists(c0: Nat) {
            satisfies_all(c0, system) and c.congr_mod(c0, system_modulus(system))
        }
} by {
    if pairwise_coprime(system_moduli(system))
        and every_modulus_positive(system)
        and satisfies_all(c, system) {
        nat_crt_list(system)
        let c0: Nat satisfy { satisfies_all(c0, system) }
        satisfies_all_unique_mod_system_modulus(system, c, c0)
        exists(solution: Nat) {
            satisfies_all(solution, system) and c.congr_mod(solution, system_modulus(system))
        }
    }
}
