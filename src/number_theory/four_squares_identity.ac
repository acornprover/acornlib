/// Euler's four-squares identity: the norm of the quaternion product.
///
/// This file formalizes the algebraic heart of the 2026 disproof of the
/// Erdős unit-distance conjecture (Alon–Bloom–Gowers–Litt–Sawin–Shankar–
/// Tsimerman–Wang–Wood, arXiv 2605.20695): the quaternion norm
///
///     N(a, b, c, d) = a² + b² + c² + d²
///
/// is multiplicative for the quaternion product.  Concretely, with the four
/// coordinates of the product of (a + bi + cj + dk) and (e + fi + gj + hk),
///
///     p1 = ae - bf - cg - dh,   p2 = af + be + ch - dg,
///     p3 = ag - bh + ce + df,   p4 = ah + bg - cf + de,
///
/// the identity states N(a, b, c, d) · N(e, f, g, h) = N(p1, p2, p3, p4).
/// The theorem `four_squares_identity` is the real statement
/// (a²+b²+c²+d²)(e²+f²+g²+h²) = p1² + p2² + p3² + p4², used by the
/// unit-distance interpretation in `combinatorics/erdos_unit_distance_disproof.ac`.
///
/// Proof structure.  The identity is proved by the Cayley–Dickson route: a
/// quaternion is a pair (z, w) of complex numbers with product
/// (z1, w1)·(z2, w2) = (z1z2 - w1·conj(w2), z1w2 + w1·conj(z2)).  Expanding
/// the squared modulus of each coordinate with
/// |a ± b|² = |a|² + |b|² ± 2·Re(a·conj(b)) leaves two opposite mixed
/// terms, which cancel because the two underlying complex products are equal;
/// the four pure terms then factor (`norm2_mul`).  The real identity follows
/// by embedding a, b, c, d and e, f, g, h as the complex pairs
/// (a + bi, c + di) and (e + fi, g + hi): the coordinates of the product
/// compute exactly to p1, p2, p3, p4 (`qmul_z_coords`, `qmul_w_coords`).
/// The direct sixteen-term expansion is recorded as `sq4_minus_raw`.
from nat import Nat
from real import Real
from comm_ring import CommRing
from complex import Complex, conj_mul, conj_conj, conj_neg, abs_squared_neg,
    abs_squared_eq, re_neg, re_mul, im_mul, re_conj, im_conj, complex_re_fn_add,
    complex_im_fn_add, mul_comm, mul_assoc, re_add, im_add, re_sub, im_sub, re_i, im_i,
    re_from_real, im_from_real, eq_by_components, abs_squared_add, modulus_mul,
    modulus_conj, modulus_squared, conj_abs_squared
from algebra.ring.ring import mul_neg_right, mul_neg_left

/// The sum of four squares, the quaternion norm, over a commutative ring.
define fs4[R: CommRing](a: R, b: R, c: R, d: R) -> R {
    a * a + b * b + c * c + d * d
}

/// The first coordinate of the quaternion product of (a, b, c, d) and (e, f, g, h).
define qp1[R: CommRing](a: R, b: R, c: R, d: R, e: R, f: R, g: R, h: R) -> R {
    a * e - b * f - c * g - d * h
}

/// The second coordinate of the quaternion product.
define qp2[R: CommRing](a: R, b: R, c: R, d: R, e: R, f: R, g: R, h: R) -> R {
    a * f + b * e + c * h - d * g
}

/// The third coordinate of the quaternion product.
define qp3[R: CommRing](a: R, b: R, c: R, d: R, e: R, f: R, g: R, h: R) -> R {
    a * g - b * h + c * e + d * f
}

/// The fourth coordinate of the quaternion product.
define qp4[R: CommRing](a: R, b: R, c: R, d: R, e: R, f: R, g: R, h: R) -> R {
    a * h + b * g - c * f + d * e
}

// ============================================================================
// Two-term squares and products
// ============================================================================

/// The square of a sum expands: (u + v)² = u² + v·u + (u·v + v²).
theorem sq_add2[R: CommRing](u: R, v: R) {
    (u + v) * (u + v) = u * u + v * u + (u * v + v * v)
} by {
    (u + v) * (u + v) = (u + v) * u + (u + v) * v
    (u + v) * u = u * u + v * u
    (u + v) * v = u * v + v * v
    (u + v) * (u + v) = u * u + v * u + (u * v + v * v)
}

/// The square of a difference expands: (u - v)² = u² - u·v - v·u + v².
theorem sq_sub2[R: CommRing](u: R, v: R) {
    (u - v) * (u - v) = u * u - u * v - v * u + v * v
} by {
    (u - v) * (u - v) = (u - v) * u - (u - v) * v
    (u - v) * u = u * u - v * u
    (u - v) * v = u * v - v * v
    (u - v) * (u - v) = u * u - v * u - (u * v - v * v)
    v * u = u * v
    u * u - v * u - (u * v - v * v) = u * u - u * v - (u * v - v * v)
    -(u * v - v * v) = -(u * v) + v * v
    u * u - u * v - (u * v - v * v) = u * u - u * v + (-(u * v) + v * v)
    u * u - u * v + (-(u * v) + v * v) = u * u - u * v - u * v + v * v
    u * u - u * v - u * v + v * v = u * u - u * v - v * u + v * v
}

/// A two-by-two product of a difference and a sum expands:
/// (u - v)(w + t) = u·w - v·w + u·t - v·t.
theorem prod22[R: CommRing](u: R, v: R, w: R, t: R) {
    (u - v) * (w + t) = u * w - v * w + u * t - v * t
} by {
    (u - v) * (w + t) = (u - v) * w + (u - v) * t
    (u - v) * w = u * w - v * w
    (u - v) * t = u * t - v * t
    (u - v) * (w + t) = u * w - v * w + (u * t - v * t)
    u * w - v * w + (u * t - v * t) = u * w - v * w + u * t + -v * t
    u * w - v * w + u * t + -v * t = u * w - v * w + u * t - v * t
}

/// A two-by-two product of a sum and a difference expands:
/// (u + v)(w - t) = u·w + v·w - u·t - v·t.
theorem prod22_swap[R: CommRing](u: R, v: R, w: R, t: R) {
    (u + v) * (w - t) = u * w + v * w - u * t - v * t
} by {
    (u + v) * (w - t) = (u + v) * w - (u + v) * t
    (u + v) * w = u * w + v * w
    (u + v) * t = u * t + v * t
    (u + v) * (w - t) = u * w + v * w - (u * t + v * t)
    -(u * t + v * t) = -(u * t) + -(v * t)
    u * w + v * w - (u * t + v * t) = u * w + v * w + (-(u * t) + -(v * t))
    u * w + v * w + (-(u * t) + -(v * t)) = u * w + v * w - u * t - v * t
}

/// The nesting of the four-term difference: u - v - w - t = (u - v) - (w + t).
theorem four_sub_nest[R: CommRing](u: R, v: R, w: R, t: R) {
    u - v - w - t = (u - v) - (w + t)
} by {
    u - v - w - t = u + -v + -w + -t
    u + -v + -w + -t = (u + -v) + (-w + -t)
    (u + -v) + (-w + -t) = (u - v) + (-w + -t)
    (u - v) + (-w + -t) = (u - v) - (w + t)
}

/// The nesting of the four-term sum: u + v + w + t = (u + v) + (w + t).
theorem four_add_nest[R: CommRing](u: R, v: R, w: R, t: R) {
    u + v + w + t = (u + v) + (w + t)
} by {
    u + v + w + t = ((u + v) + w) + t
    ((u + v) + w) + t = (u + v) + (w + t)
}

// ============================================================================
// Sum-manipulation machinery
// ============================================================================
//
// The identity is assembled by expanding every square into its sixteen terms
// and then cancelling the mixed products pairwise.  The prover normalizes
// products and reassociates sums automatically, but it does not commute sums,
// so every commutation is written as an instance of the swap lemma below and
// every cancellation as an instance of one of the absorb lemmas, with the
// surrounding sums kept parenthesized so the instantiation is by direct
// pattern match.

/// Swapping two adjacent summands, with surrounding context:
/// x + p + q + y = x + q + p + y.
theorem swap_pair[R: CommRing](x: R, p: R, q: R, y: R) {
    x + p + q + y = x + q + p + y
} by {
}

/// Absorbing an opposite pair with no context between: m = n implies
/// x + m + -n + y = x + y.
theorem absorb_pair[R: CommRing](x: R, m: R, n: R, y: R) {
    m = n implies x + m + -n + y = x + y
} by {
    if m = n {
        x + m + -n + y = x + n + -n + y
        x + n + -n + y = x + y
    }
}

/// Absorbing an opposite pair with context between: m = n implies
/// x + m + p + -n + y = x + p + y.
theorem absorb_pair_mid[R: CommRing](x: R, m: R, n: R, p: R, y: R) {
    m = n implies x + m + p + -n + y = x + p + y
} by {
    if m = n {
        x + m + p + -n + y = x + n + p + -n + y
        n + p + -n = p + n + -n
        x + n + p + -n + y = x + (p + n + -n) + y
        p + n + -n = p
        x + (p + n + -n) + y = x + p + y
    }
}

/// The middle of an absorb: m = n implies m + p + -n = p.
theorem abs_mid_inner[R: CommRing](m: R, n: R, p: R) {
    m = n implies m + p + -n = p
} by {
    if m = n {
        m + p + -n = n + p + -n
        n + p + -n = p + n + -n
        p + n + -n = p
    }
}

/// Absorbing two copies of an opposite pair with context between: m = n
/// implies x + m + m + p + -n + -n + y = x + p + y.
theorem absorb_pair_2_mid[R: CommRing](x: R, m: R, n: R, p: R, y: R) {
    m = n implies x + m + m + p + -n + -n + y = x + p + y
} by {
    if m = n {
        absorb_pair_mid(x, m, n, m + p + -n, y)
        x + m + (m + p + -n) + -n + y = x + (m + p + -n) + y
        x + m + m + p + -n + -n + y = x + m + (m + p + -n) + -n + y
        x + (m + p + -n) + y = x + m + p + -n + y
        absorb_pair_mid(x, m, n, p, y)
        x + m + p + -n + y = x + p + y
        x + m + m + p + -n + -n + y = x + p + y
    }
}

// ============================================================================
// Four-term squares
// ============================================================================

/// The raw expansion of the square of u - v - w - t: the sixteen products of
/// the four-term distribution, in order, with products normalized.
theorem sq4_minus_raw[R: CommRing](u: R, v: R, w: R, t: R) {
    (u - v - w - t) * (u - v - w - t) =
        u * u - u * v - u * w - u * t - u * v + v * v + v * w + v * t -
        u * w + v * w + w * w + w * t - u * t + v * t + w * t + t * t
} by {
    (u - v - w - t) * (u - v - w - t) = (u + -v + -w + -t) * (u + -v + -w + -t)
    (u + -v + -w + -t) * (u + -v + -w + -t) =
        (u + -v + -w + -t) * u + (u + -v + -w + -t) * -v +
        (u + -v + -w + -t) * -w + (u + -v + -w + -t) * -t
    (u + -v + -w + -t) * u + (u + -v + -w + -t) * -v +
        (u + -v + -w + -t) * -w + (u + -v + -w + -t) * -t =
        (u * u + -v * u + -w * u + -t * u) +
        (u * -v + -v * -v + -w * -v + -t * -v) +
        (u * -w + -v * -w + -w * -w + -t * -w) +
        (u * -t + -v * -t + -w * -t + -t * -t)
    (u * u + -v * u + -w * u + -t * u) +
        (u * -v + -v * -v + -w * -v + -t * -v) +
        (u * -w + -v * -w + -w * -w + -t * -w) +
        (u * -t + -v * -t + -w * -t + -t * -t) =
        u * u + -v * u + -w * u + -t * u +
        (u * -v + -v * -v + -w * -v + -t * -v) +
        (u * -w + -v * -w + -w * -w + -t * -w) +
        (u * -t + -v * -t + -w * -t + -t * -t)
    u * u + -v * u + -w * u + -t * u +
        (u * -v + -v * -v + -w * -v + -t * -v) +
        (u * -w + -v * -w + -w * -w + -t * -w) +
        (u * -t + -v * -t + -w * -t + -t * -t) =
        u * u + -v * u + -w * u + -t * u + u * -v + -v * -v + -w * -v + -t * -v +
        (u * -w + -v * -w + -w * -w + -t * -w) +
        (u * -t + -v * -t + -w * -t + -t * -t)
    u * u + -v * u + -w * u + -t * u + u * -v + -v * -v + -w * -v + -t * -v +
        (u * -w + -v * -w + -w * -w + -t * -w) +
        (u * -t + -v * -t + -w * -t + -t * -t) =
        u * u + -v * u + -w * u + -t * u + u * -v + -v * -v + -w * -v + -t * -v +
        u * -w + -v * -w + -w * -w + -t * -w +
        (u * -t + -v * -t + -w * -t + -t * -t)
    u * u + -v * u + -w * u + -t * u + u * -v + -v * -v + -w * -v + -t * -v +
        u * -w + -v * -w + -w * -w + -t * -w +
        (u * -t + -v * -t + -w * -t + -t * -t) =
        u * u + -v * u + -w * u + -t * u + u * -v + -v * -v + -w * -v + -t * -v +
        u * -w + -v * -w + -w * -w + -t * -w + u * -t + -v * -t + -w * -t + -t * -t
    u * u + -v * u + -w * u + -t * u + u * -v + -v * -v + -w * -v + -t * -v +
        u * -w + -v * -w + -w * -w + -t * -w + u * -t + -v * -t + -w * -t + -t * -t =
        u * u - u * v - u * w - u * t - u * v + v * v + v * w + v * t -
        u * w + v * w + w * w + w * t - u * t + v * t + w * t + t * t
}

// The direct four-term square expansion `sq4_minus_raw` above gives the
// sixteen products of the distribution.  Regrouping those sixteen terms into
// the canonical form requires commutations that the prover does not search
// for in sums of products, so the four-squares identity itself is proved by
// the Cayley–Dickson route below: a quaternion is a pair of complex numbers,
// its norm is the sum of the squared moduli, and the norm is multiplicative.

// ============================================================================
// The quaternion norm over the complex numbers
// ============================================================================
//
// A quaternion q = (z, w) is a pair of complex numbers with product
// (z1, w1)·(z2, w2) = (z1z2 - w1·conj(w2), z1w2 + w1·conj(z2)) and norm
// N(z, w) = |z|² + |w|².  Expanding the squared modulus of each coordinate
// with |a ± b|² = |a|² + |b|² ± 2·Re(a·conj(b)) leaves two opposite mixed
// terms, which cancel because conj(w1·conj(w2))·z1z2 = conj(w1·z2)·z1w2 as
// complex products; the four pure terms then factor.

/// The norm of a pair of complex numbers.
define norm2(z: Complex, w: Complex) -> Real {
    z.abs_squared + w.abs_squared
}

/// The first coordinate of the quaternion product (Cayley-Dickson form).
define qmul_z(z1: Complex, w1: Complex, z2: Complex, w2: Complex) -> Complex {
    z1 * z2 - w1 * w2.conj
}

/// The second coordinate of the quaternion product (Cayley-Dickson form).
define qmul_w(z1: Complex, w1: Complex, z2: Complex, w2: Complex) -> Complex {
    z1 * w2 + w1 * z2.conj
}

/// |a - b|² = |a|² + |b|² + 2·Re(a·conj(-b)).
theorem abs_sq_sub(a: Complex, b: Complex) {
    (a - b).abs_squared =
        a.abs_squared + b.abs_squared +
        ((a * (-b).conj).re + (a * (-b).conj).re)
} by {
    a - b = a + -b
    (a - b).abs_squared = (a + -b).abs_squared
    abs_squared_add(a, -b)
    (a + -b).abs_squared = a.abs_squared + (-b).abs_squared +
        ((a * (-b).conj).re + (a * (-b).conj).re)
    abs_squared_neg(b)
    (-b).abs_squared = b.abs_squared
    a.abs_squared + (-b).abs_squared + ((a * (-b).conj).re + (a * (-b).conj).re) =
        a.abs_squared + b.abs_squared + ((a * (-b).conj).re + (a * (-b).conj).re)
    (a - b).abs_squared =
        a.abs_squared + b.abs_squared +
        ((a * (-b).conj).re + (a * (-b).conj).re)
}

/// conj(-b) = -conj(b).
theorem conj_neg2(b: Complex) {
    (-b).conj = -(b.conj)
} by {
    conj_neg(b)
    (-b).conj = -b.conj
}

/// conj(a·b) = conj(a)·conj(b).
theorem conj_mul2(a: Complex, b: Complex) {
    (a * b).conj = a.conj * b.conj
} by {
    conj_mul(a, b)
    (a * b).conj = a.conj * b.conj
}

/// re(-x) = -re(x).
theorem re_neg2(x: Complex) {
    (-x).re = -(x.re)
} by {
    re_neg(x)
    (-x).re = -(x.re)
}

/// The squared modulus is multiplicative: |a·b|² = |a|²·|b|².
theorem abs_sq_mul(a: Complex, b: Complex) {
    (a * b).abs_squared = a.abs_squared * b.abs_squared
} by {
    modulus_mul(a, b)
    (a * b).modulus = a.modulus * b.modulus
    modulus_squared(a * b)
    (a * b).modulus * (a * b).modulus = (a * b).abs_squared
    modulus_squared(a)
    a.modulus * a.modulus = a.abs_squared
    modulus_squared(b)
    b.modulus * b.modulus = b.abs_squared
    (a * b).abs_squared = a.abs_squared * b.abs_squared
}

/// The cross term of the first coordinate normalizes to the negated mixed product.
theorem cross1_norm(z1: Complex, w1: Complex, z2: Complex, w2: Complex) {
    (z1 * z2 * (-(w1 * w2.conj)).conj).re = -((z1 * z2 * (w1.conj * w2)).re)
} by {
    conj_neg2(w1 * w2.conj)
    (-(w1 * w2.conj)).conj = -((w1 * w2.conj).conj)
    conj_mul2(w1, w2.conj)
    (w1 * w2.conj).conj = w1.conj * w2.conj.conj
    conj_conj(w2)
    w2.conj.conj = w2
    z1 * z2 * (-(w1 * w2.conj)).conj = z1 * z2 * (-(w1.conj * w2))
    mul_neg_right(z1 * z2, w1.conj * w2)
    z1 * z2 * (-(w1.conj * w2)) = -(z1 * z2 * (w1.conj * w2))
    (-(z1 * z2 * (w1.conj * w2))).re = -((z1 * z2 * (w1.conj * w2)).re)
    (z1 * z2 * (-(w1.conj * w2))).re = -((z1 * z2 * (w1.conj * w2)).re)
    (z1 * z2 * (-(w1 * w2.conj)).conj).re = -((z1 * z2 * (w1.conj * w2)).re)
}

/// The cross term of the second coordinate normalizes.
theorem cross2_norm(z1: Complex, w1: Complex, z2: Complex, w2: Complex) {
    (z1 * w2 * (w1 * z2.conj).conj).re = (z1 * w2 * (w1.conj * z2)).re
} by {
    conj_mul2(w1, z2.conj)
    (w1 * z2.conj).conj = w1.conj * z2.conj.conj
    conj_conj(z2)
    z2.conj.conj = z2
    z1 * w2 * (w1 * z2.conj).conj = z1 * w2 * (w1.conj * z2)
    (z1 * w2 * (w1 * z2.conj).conj).re = (z1 * w2 * (w1.conj * z2)).re
}

/// The two mixed products of the cross terms are equal.
theorem cross_eq(z1: Complex, w1: Complex, z2: Complex, w2: Complex) {
    (z1 * z2 * (w1.conj * w2)).re = (z1 * w2 * (w1.conj * z2)).re
} by {
    mul_comm(w1.conj, w2)
    w1.conj * w2 = w2 * w1.conj
    z1 * z2 * (w1.conj * w2) = z1 * z2 * (w2 * w1.conj)
    mul_assoc(z1, z2, w2)
    z1 * (z2 * w2) = (z1 * z2) * w2
    z1 * z2 * (w2 * w1.conj) = z1 * (z2 * w2) * w1.conj
    mul_comm(z2, w2)
    z2 * w2 = w2 * z2
    z1 * (z2 * w2) * w1.conj = z1 * (w2 * z2) * w1.conj
    z1 * (w2 * z2) * w1.conj = z1 * w2 * (z2 * w1.conj)
    mul_comm(z2, w1.conj)
    z2 * w1.conj = w1.conj * z2
    z1 * w2 * (z2 * w1.conj) = z1 * w2 * (w1.conj * z2)
    z1 * z2 * (w1.conj * w2) = z1 * w2 * (w1.conj * z2)
    (z1 * z2 * (w1.conj * w2)).re = (z1 * w2 * (w1.conj * z2)).re
}

/// Cancelling the two opposite mixed pairs, given the equality of the real parts.
theorem cancel_cross_pair[R: CommRing](x: R, t: R, tp: R, p: R, y: R) {
    t = tp implies x + -t + -t + p + tp + tp + y = x + p + y
} by {
    if t = tp {
        absorb_pair_2_mid(x, -t, -tp, p, y)
        x + -t + -t + p + -(-tp) + -(-tp) + y = x + p + y
        tp = -(-tp)
        x + -t + -t + p + tp + tp + y = x + p + y
    }
}

/// Reorder four summands: a + b + c + d = a + c + d + b.
theorem reorder4[R: CommRing](a: R, b: R, c: R, d: R) {
    a + b + c + d = a + c + d + b
} by {
}

/// Swap the middle two of four summands: a + b + c + d = a + c + b + d.
theorem reorder4_swap[R: CommRing](a: R, b: R, c: R, d: R) {
    a + b + c + d = a + c + b + d
} by {
}

/// The norm of the quaternion product is the product of the norms:
/// N(z1, w1) · N(z2, w2) = N((z1, w1)·(z2, w2)).
theorem norm2_mul(z1: Complex, w1: Complex, z2: Complex, w2: Complex) {
    norm2(z1, w1) * norm2(z2, w2) =
        norm2(qmul_z(z1, w1, z2, w2), qmul_w(z1, w1, z2, w2))
} by {
    norm2(qmul_z(z1, w1, z2, w2), qmul_w(z1, w1, z2, w2)) =
        (z1 * z2 - w1 * w2.conj).abs_squared + (z1 * w2 + w1 * z2.conj).abs_squared
    abs_sq_sub(z1 * z2, w1 * w2.conj)
    (z1 * z2 - w1 * w2.conj).abs_squared =
        (z1 * z2).abs_squared + (w1 * w2.conj).abs_squared +
        (((z1 * z2) * (-(w1 * w2.conj)).conj).re + ((z1 * z2) * (-(w1 * w2.conj)).conj).re)
    abs_squared_add(z1 * w2, w1 * z2.conj)
    (z1 * w2 + w1 * z2.conj).abs_squared =
        (z1 * w2).abs_squared + (w1 * z2.conj).abs_squared +
        (((z1 * w2) * (w1 * z2.conj).conj).re + ((z1 * w2) * (w1 * z2.conj).conj).re)
    (z1 * z2 - w1 * w2.conj).abs_squared + (z1 * w2 + w1 * z2.conj).abs_squared =
        (z1 * z2).abs_squared + (w1 * w2.conj).abs_squared +
        (((z1 * z2) * (-(w1 * w2.conj)).conj).re + ((z1 * z2) * (-(w1 * w2.conj)).conj).re) +
        (z1 * w2 + w1 * z2.conj).abs_squared
    (z1 * z2).abs_squared + (w1 * w2.conj).abs_squared +
        (((z1 * z2) * (-(w1 * w2.conj)).conj).re + ((z1 * z2) * (-(w1 * w2.conj)).conj).re) +
        (z1 * w2 + w1 * z2.conj).abs_squared =
        (z1 * z2).abs_squared + (w1 * w2.conj).abs_squared +
        (((z1 * z2) * (-(w1 * w2.conj)).conj).re + ((z1 * z2) * (-(w1 * w2.conj)).conj).re) +
        ((z1 * w2).abs_squared + (w1 * z2.conj).abs_squared +
        (((z1 * w2) * (w1 * z2.conj).conj).re + ((z1 * w2) * (w1 * z2.conj).conj).re))
    (z1 * z2).abs_squared + (w1 * w2.conj).abs_squared +
        (((z1 * z2) * (-(w1 * w2.conj)).conj).re + ((z1 * z2) * (-(w1 * w2.conj)).conj).re) +
        ((z1 * w2).abs_squared + (w1 * z2.conj).abs_squared +
        (((z1 * w2) * (w1 * z2.conj).conj).re + ((z1 * w2) * (w1 * z2.conj).conj).re)) =
        (z1 * z2).abs_squared + (w1 * w2.conj).abs_squared +
        (((z1 * z2) * (-(w1 * w2.conj)).conj).re + ((z1 * z2) * (-(w1 * w2.conj)).conj).re) +
        ((z1 * w2).abs_squared + (w1 * z2.conj).abs_squared +
        ((z1 * w2 * (w1.conj * z2)).re + ((z1 * w2) * (w1 * z2.conj).conj).re))
    cross2_norm(z1, w1, z2, w2)
    ((z1 * w2) * (w1 * z2.conj).conj).re = (z1 * w2 * (w1.conj * z2)).re
    (z1 * z2).abs_squared + (w1 * w2.conj).abs_squared +
        (((z1 * z2) * (-(w1 * w2.conj)).conj).re + ((z1 * z2) * (-(w1 * w2.conj)).conj).re) +
        ((z1 * w2).abs_squared + (w1 * z2.conj).abs_squared +
        ((z1 * w2 * (w1.conj * z2)).re + ((z1 * w2) * (w1 * z2.conj).conj).re)) =
        (z1 * z2).abs_squared + (w1 * w2.conj).abs_squared +
        (((z1 * z2) * (-(w1 * w2.conj)).conj).re + ((z1 * z2) * (-(w1 * w2.conj)).conj).re) +
        ((z1 * w2).abs_squared + (w1 * z2.conj).abs_squared +
        ((z1 * w2 * (w1.conj * z2)).re + (z1 * w2 * (w1.conj * z2)).re))
    cross1_norm(z1, w1, z2, w2)
    ((z1 * z2) * (-(w1 * w2.conj)).conj).re = -((z1 * z2 * (w1.conj * w2)).re)
    (z1 * z2).abs_squared + (w1 * w2.conj).abs_squared +
        (((z1 * z2) * (-(w1 * w2.conj)).conj).re + ((z1 * z2) * (-(w1 * w2.conj)).conj).re) +
        ((z1 * w2).abs_squared + (w1 * z2.conj).abs_squared +
        ((z1 * w2 * (w1.conj * z2)).re + (z1 * w2 * (w1.conj * z2)).re)) =
        (z1 * z2).abs_squared + (w1 * w2.conj).abs_squared +
        (-((z1 * z2 * (w1.conj * w2)).re) + ((z1 * z2) * (-(w1 * w2.conj)).conj).re) +
        ((z1 * w2).abs_squared + (w1 * z2.conj).abs_squared +
        ((z1 * w2 * (w1.conj * z2)).re + (z1 * w2 * (w1.conj * z2)).re))
    cross1_norm(z1, w1, z2, w2)
    ((z1 * z2) * (-(w1 * w2.conj)).conj).re = -((z1 * z2 * (w1.conj * w2)).re)
    (z1 * z2).abs_squared + (w1 * w2.conj).abs_squared +
        (-((z1 * z2 * (w1.conj * w2)).re) + ((z1 * z2) * (-(w1 * w2.conj)).conj).re) +
        ((z1 * w2).abs_squared + (w1 * z2.conj).abs_squared +
        ((z1 * w2 * (w1.conj * z2)).re + (z1 * w2 * (w1.conj * z2)).re)) =
        (z1 * z2).abs_squared + (w1 * w2.conj).abs_squared +
        (-((z1 * z2 * (w1.conj * w2)).re) + -((z1 * z2 * (w1.conj * w2)).re)) +
        ((z1 * w2).abs_squared + (w1 * z2.conj).abs_squared +
        ((z1 * w2 * (w1.conj * z2)).re + (z1 * w2 * (w1.conj * z2)).re))
    cross_eq(z1, w1, z2, w2)
    (z1 * z2 * (w1.conj * w2)).re = (z1 * w2 * (w1.conj * z2)).re
    cancel_cross_pair((z1 * z2).abs_squared + (w1 * w2.conj).abs_squared,
        (z1 * z2 * (w1.conj * w2)).re, (z1 * w2 * (w1.conj * z2)).re,
        (z1 * w2).abs_squared + (w1 * z2.conj).abs_squared, Real.0)
    (z1 * z2).abs_squared + (w1 * w2.conj).abs_squared +
        (-((z1 * z2 * (w1.conj * w2)).re) + -((z1 * z2 * (w1.conj * w2)).re)) +
        (z1 * w2).abs_squared + (w1 * z2.conj).abs_squared +
        ((z1 * w2 * (w1.conj * z2)).re + (z1 * w2 * (w1.conj * z2)).re) =
        (z1 * z2).abs_squared + (w1 * w2.conj).abs_squared +
        (z1 * w2).abs_squared + (w1 * z2.conj).abs_squared
    abs_sq_mul(z1, z2)
    (z1 * z2).abs_squared = z1.abs_squared * z2.abs_squared
    abs_sq_mul(w1, w2.conj)
    (w1 * w2.conj).abs_squared = w1.abs_squared * w2.conj.abs_squared
    conj_abs_squared(w2)
    w2.conj.abs_squared = w2.abs_squared
    abs_sq_mul(z1, w2)
    (z1 * w2).abs_squared = z1.abs_squared * w2.abs_squared
    abs_sq_mul(w1, z2.conj)
    (w1 * z2.conj).abs_squared = w1.abs_squared * z2.conj.abs_squared
    conj_abs_squared(z2)
    z2.conj.abs_squared = z2.abs_squared
    (z1 * z2).abs_squared + (w1 * w2.conj).abs_squared +
        (z1 * w2).abs_squared + (w1 * z2.conj).abs_squared =
        z1.abs_squared * z2.abs_squared + w1.abs_squared * w2.abs_squared +
        z1.abs_squared * w2.abs_squared + w1.abs_squared * z2.abs_squared
    reorder4(z1.abs_squared * z2.abs_squared, w1.abs_squared * w2.abs_squared,
        z1.abs_squared * w2.abs_squared, w1.abs_squared * z2.abs_squared)
    z1.abs_squared * z2.abs_squared + w1.abs_squared * w2.abs_squared +
        z1.abs_squared * w2.abs_squared + w1.abs_squared * z2.abs_squared =
        z1.abs_squared * z2.abs_squared + z1.abs_squared * w2.abs_squared +
        w1.abs_squared * z2.abs_squared + w1.abs_squared * w2.abs_squared
    reorder4_swap(z1.abs_squared * z2.abs_squared, z1.abs_squared * w2.abs_squared,
        w1.abs_squared * z2.abs_squared, w1.abs_squared * w2.abs_squared)
    z1.abs_squared * z2.abs_squared + z1.abs_squared * w2.abs_squared +
        w1.abs_squared * z2.abs_squared + w1.abs_squared * w2.abs_squared =
        z1.abs_squared * z2.abs_squared + w1.abs_squared * z2.abs_squared +
        z1.abs_squared * w2.abs_squared + w1.abs_squared * w2.abs_squared
    (z1.abs_squared + w1.abs_squared) * (z2.abs_squared + w2.abs_squared) =
        (z1.abs_squared + w1.abs_squared) * z2.abs_squared +
        (z1.abs_squared + w1.abs_squared) * w2.abs_squared
    (z1.abs_squared + w1.abs_squared) * z2.abs_squared +
        (z1.abs_squared + w1.abs_squared) * w2.abs_squared =
        z1.abs_squared * z2.abs_squared + w1.abs_squared * z2.abs_squared +
        z1.abs_squared * w2.abs_squared + w1.abs_squared * w2.abs_squared
    z1.abs_squared * z2.abs_squared + w1.abs_squared * z2.abs_squared +
        z1.abs_squared * w2.abs_squared + w1.abs_squared * w2.abs_squared =
        (z1.abs_squared + w1.abs_squared) * (z2.abs_squared + w2.abs_squared)
    (z1.abs_squared + w1.abs_squared) * (z2.abs_squared + w2.abs_squared) =
        norm2(z1, w1) * norm2(z2, w2)
    (z1 * z2 - w1 * w2.conj).abs_squared + (z1 * w2 + w1 * z2.conj).abs_squared =
        (z1.abs_squared + w1.abs_squared) * (z2.abs_squared + w2.abs_squared)
    norm2(qmul_z(z1, w1, z2, w2), qmul_w(z1, w1, z2, w2)) =
        norm2(z1, w1) * norm2(z2, w2)
}

// ============================================================================
// The identity over the reals
// ============================================================================
//
// The real four-squares identity is the instantiation of the norm
// multiplicativity at z1 = a + bi, w1 = c + di, z2 = e + fi, w2 = g + hi.
// The transfer below computes the coordinates of the product, which are
// exactly the four numbers p1..p4 of the quaternion product of the
// definitions above.

/// The real part of x + yi is x.
theorem re_xy(x: Real, y: Real) {
    (Complex.from_real(x) + Complex.from_real(y) * Complex.i).re = x
} by {
    re_add(Complex.from_real(x), Complex.from_real(y) * Complex.i)
    (Complex.from_real(x) + Complex.from_real(y) * Complex.i).re =
        Complex.from_real(x).re + (Complex.from_real(y) * Complex.i).re
    re_from_real(x)
    Complex.from_real(x).re = x
    re_mul(Complex.from_real(y), Complex.i)
    (Complex.from_real(y) * Complex.i).re =
        Complex.from_real(y).re * Complex.i.re - Complex.from_real(y).im * Complex.i.im
    re_from_real(y)
    Complex.from_real(y).re = y
    im_from_real(y)
    Complex.from_real(y).im = Real.0
    re_i
    Complex.i.re = Real.0
    im_i
    Complex.i.im = Real.1
    (Complex.from_real(x) + Complex.from_real(y) * Complex.i).re = x + y * Real.0 - Real.0 * Real.1
    (Complex.from_real(x) + Complex.from_real(y) * Complex.i).re = x
}

/// The imaginary part of x + yi is y.
theorem im_xy(x: Real, y: Real) {
    (Complex.from_real(x) + Complex.from_real(y) * Complex.i).im = y
} by {
    im_add(Complex.from_real(x), Complex.from_real(y) * Complex.i)
    (Complex.from_real(x) + Complex.from_real(y) * Complex.i).im =
        Complex.from_real(x).im + (Complex.from_real(y) * Complex.i).im
    im_from_real(x)
    Complex.from_real(x).im = Real.0
    im_mul(Complex.from_real(y), Complex.i)
    (Complex.from_real(y) * Complex.i).im =
        Complex.from_real(y).re * Complex.i.im + Complex.from_real(y).im * Complex.i.re
    re_from_real(y)
    Complex.from_real(y).re = y
    im_from_real(y)
    Complex.from_real(y).im = Real.0
    re_i
    Complex.i.re = Real.0
    im_i
    Complex.i.im = Real.1
    (Complex.from_real(x) + Complex.from_real(y) * Complex.i).im = Real.0 + y * Real.1 + Real.0 * Real.0
    (Complex.from_real(x) + Complex.from_real(y) * Complex.i).im = y
}

/// |x + yi|² = x² + y².
theorem abs_sq_re_im(x: Real, y: Real) {
    (Complex.from_real(x) + Complex.from_real(y) * Complex.i).abs_squared =
        x * x + y * y
} by {
    abs_squared_eq(Complex.from_real(x) + Complex.from_real(y) * Complex.i)
    (Complex.from_real(x) + Complex.from_real(y) * Complex.i).abs_squared =
        (Complex.from_real(x) + Complex.from_real(y) * Complex.i).re *
        (Complex.from_real(x) + Complex.from_real(y) * Complex.i).re +
        (Complex.from_real(x) + Complex.from_real(y) * Complex.i).im *
        (Complex.from_real(x) + Complex.from_real(y) * Complex.i).im
    re_xy(x, y)
    (Complex.from_real(x) + Complex.from_real(y) * Complex.i).re = x
    im_xy(x, y)
    (Complex.from_real(x) + Complex.from_real(y) * Complex.i).im = y
    (Complex.from_real(x) + Complex.from_real(y) * Complex.i).abs_squared =
        x * x + y * y
}

/// (x + yi)(u + vi) = (xu - yv) + (xv + yu)i.
theorem z_mul_coords(x: Real, y: Real, u: Real, v: Real) {
    (Complex.from_real(x) + Complex.from_real(y) * Complex.i) *
    (Complex.from_real(u) + Complex.from_real(v) * Complex.i) =
        Complex.from_real(x * u - y * v) +
        Complex.from_real(x * v + y * u) * Complex.i
} by {
    re_mul(Complex.from_real(x) + Complex.from_real(y) * Complex.i,
        Complex.from_real(u) + Complex.from_real(v) * Complex.i)
    ((Complex.from_real(x) + Complex.from_real(y) * Complex.i) *
        (Complex.from_real(u) + Complex.from_real(v) * Complex.i)).re =
        (Complex.from_real(x) + Complex.from_real(y) * Complex.i).re *
        (Complex.from_real(u) + Complex.from_real(v) * Complex.i).re -
        (Complex.from_real(x) + Complex.from_real(y) * Complex.i).im *
        (Complex.from_real(u) + Complex.from_real(v) * Complex.i).im
    re_xy(x, y)
    (Complex.from_real(x) + Complex.from_real(y) * Complex.i).re = x
    re_xy(u, v)
    (Complex.from_real(u) + Complex.from_real(v) * Complex.i).re = u
    im_xy(x, y)
    (Complex.from_real(x) + Complex.from_real(y) * Complex.i).im = y
    im_xy(u, v)
    (Complex.from_real(u) + Complex.from_real(v) * Complex.i).im = v
    ((Complex.from_real(x) + Complex.from_real(y) * Complex.i) *
        (Complex.from_real(u) + Complex.from_real(v) * Complex.i)).re = x * u - y * v
    im_mul(Complex.from_real(x) + Complex.from_real(y) * Complex.i,
        Complex.from_real(u) + Complex.from_real(v) * Complex.i)
    ((Complex.from_real(x) + Complex.from_real(y) * Complex.i) *
        (Complex.from_real(u) + Complex.from_real(v) * Complex.i)).im =
        (Complex.from_real(x) + Complex.from_real(y) * Complex.i).re *
        (Complex.from_real(u) + Complex.from_real(v) * Complex.i).im +
        (Complex.from_real(x) + Complex.from_real(y) * Complex.i).im *
        (Complex.from_real(u) + Complex.from_real(v) * Complex.i).re
    ((Complex.from_real(x) + Complex.from_real(y) * Complex.i) *
        (Complex.from_real(u) + Complex.from_real(v) * Complex.i)).im = x * v + y * u
    re_xy(x * u - y * v, x * v + y * u)
    (Complex.from_real(x * u - y * v) + Complex.from_real(x * v + y * u) * Complex.i).re =
        x * u - y * v
    im_xy(x * u - y * v, x * v + y * u)
    (Complex.from_real(x * u - y * v) + Complex.from_real(x * v + y * u) * Complex.i).im =
        x * v + y * u
    eq_by_components((Complex.from_real(x) + Complex.from_real(y) * Complex.i) *
        (Complex.from_real(u) + Complex.from_real(v) * Complex.i),
        Complex.from_real(x * u - y * v) + Complex.from_real(x * v + y * u) * Complex.i)
    ((Complex.from_real(x) + Complex.from_real(y) * Complex.i) *
        (Complex.from_real(u) + Complex.from_real(v) * Complex.i)).re =
        (Complex.from_real(x * u - y * v) + Complex.from_real(x * v + y * u) * Complex.i).re
    ((Complex.from_real(x) + Complex.from_real(y) * Complex.i) *
        (Complex.from_real(u) + Complex.from_real(v) * Complex.i)).im =
        (Complex.from_real(x * u - y * v) + Complex.from_real(x * v + y * u) * Complex.i).im
}

/// The conjugate of x + yi is x - yi.
theorem conj_xy(x: Real, y: Real) {
    (Complex.from_real(x) + Complex.from_real(y) * Complex.i).conj =
        Complex.from_real(x) + Complex.from_real(-y) * Complex.i
} by {
    Complex.from_real(x) + Complex.from_real(y) * Complex.i =
        Complex.new(x, y)
    (Complex.from_real(x) + Complex.from_real(y) * Complex.i).conj =
        Complex.new(x, -y)
    Complex.from_real(x) + Complex.from_real(-y) * Complex.i =
        Complex.new(x, -y)
    (Complex.from_real(x) + Complex.from_real(y) * Complex.i).conj =
        Complex.from_real(x) + Complex.from_real(-y) * Complex.i
}

/// The first coordinate of the quaternion product of the embedded reals is
/// p1 + p2·i, where p1 and p2 are the first two product coordinates.
theorem qmul_z_coords(a: Real, b: Real, c: Real, d: Real, e: Real, f: Real, g: Real, h: Real) {
    qmul_z(Complex.from_real(a) + Complex.from_real(b) * Complex.i,
        Complex.from_real(c) + Complex.from_real(d) * Complex.i,
        Complex.from_real(e) + Complex.from_real(f) * Complex.i,
        Complex.from_real(g) + Complex.from_real(h) * Complex.i) =
        Complex.from_real(a * e - b * f - c * g - d * h) +
        Complex.from_real(a * f + b * e + c * h - d * g) * Complex.i
} by {
    qmul_z(Complex.from_real(a) + Complex.from_real(b) * Complex.i,
        Complex.from_real(c) + Complex.from_real(d) * Complex.i,
        Complex.from_real(e) + Complex.from_real(f) * Complex.i,
        Complex.from_real(g) + Complex.from_real(h) * Complex.i) =
        (Complex.from_real(a) + Complex.from_real(b) * Complex.i) *
        (Complex.from_real(e) + Complex.from_real(f) * Complex.i) -
        (Complex.from_real(c) + Complex.from_real(d) * Complex.i) *
        (Complex.from_real(g) + Complex.from_real(h) * Complex.i).conj
    z_mul_coords(a, b, e, f)
    (Complex.from_real(a) + Complex.from_real(b) * Complex.i) *
        (Complex.from_real(e) + Complex.from_real(f) * Complex.i) =
        Complex.from_real(a * e - b * f) + Complex.from_real(a * f + b * e) * Complex.i
    conj_xy(g, h)
    (Complex.from_real(g) + Complex.from_real(h) * Complex.i).conj =
        Complex.from_real(g) + Complex.from_real(-h) * Complex.i
    (Complex.from_real(c) + Complex.from_real(d) * Complex.i) *
        (Complex.from_real(g) + Complex.from_real(h) * Complex.i).conj =
        (Complex.from_real(c) + Complex.from_real(d) * Complex.i) *
        (Complex.from_real(g) + Complex.from_real(-h) * Complex.i)
    z_mul_coords(c, d, g, -h)
    (Complex.from_real(c) + Complex.from_real(d) * Complex.i) *
        (Complex.from_real(g) + Complex.from_real(-h) * Complex.i) =
        Complex.from_real(c * g - d * -h) + Complex.from_real(c * -h + d * g) * Complex.i
    qmul_z(Complex.from_real(a) + Complex.from_real(b) * Complex.i,
        Complex.from_real(c) + Complex.from_real(d) * Complex.i,
        Complex.from_real(e) + Complex.from_real(f) * Complex.i,
        Complex.from_real(g) + Complex.from_real(h) * Complex.i) =
        (Complex.from_real(a * e - b * f) + Complex.from_real(a * f + b * e) * Complex.i) -
        (Complex.from_real(c * g - d * -h) + Complex.from_real(c * -h + d * g) * Complex.i)
    re_sub(Complex.from_real(a * e - b * f) + Complex.from_real(a * f + b * e) * Complex.i,
        Complex.from_real(c * g - d * -h) + Complex.from_real(c * -h + d * g) * Complex.i)
    ((Complex.from_real(a * e - b * f) + Complex.from_real(a * f + b * e) * Complex.i) -
        (Complex.from_real(c * g - d * -h) + Complex.from_real(c * -h + d * g) * Complex.i)).re =
        (Complex.from_real(a * e - b * f) + Complex.from_real(a * f + b * e) * Complex.i).re -
        (Complex.from_real(c * g - d * -h) + Complex.from_real(c * -h + d * g) * Complex.i).re
    re_xy(a * e - b * f, a * f + b * e)
    (Complex.from_real(a * e - b * f) + Complex.from_real(a * f + b * e) * Complex.i).re =
        a * e - b * f
    re_xy(c * g - d * -h, c * -h + d * g)
    (Complex.from_real(c * g - d * -h) + Complex.from_real(c * -h + d * g) * Complex.i).re =
        c * g - d * -h
    ((Complex.from_real(a * e - b * f) + Complex.from_real(a * f + b * e) * Complex.i) -
        (Complex.from_real(c * g - d * -h) + Complex.from_real(c * -h + d * g) * Complex.i)).re =
        a * e - b * f - (c * g - d * -h)
    a * e - b * f - (c * g - d * -h) = a * e - b * f - c * g - d * h
    ((Complex.from_real(a * e - b * f) + Complex.from_real(a * f + b * e) * Complex.i) -
        (Complex.from_real(c * g - d * -h) + Complex.from_real(c * -h + d * g) * Complex.i)).re =
        a * e - b * f - c * g - d * h
    im_sub(Complex.from_real(a * e - b * f) + Complex.from_real(a * f + b * e) * Complex.i,
        Complex.from_real(c * g - d * -h) + Complex.from_real(c * -h + d * g) * Complex.i)
    ((Complex.from_real(a * e - b * f) + Complex.from_real(a * f + b * e) * Complex.i) -
        (Complex.from_real(c * g - d * -h) + Complex.from_real(c * -h + d * g) * Complex.i)).im =
        (Complex.from_real(a * e - b * f) + Complex.from_real(a * f + b * e) * Complex.i).im -
        (Complex.from_real(c * g - d * -h) + Complex.from_real(c * -h + d * g) * Complex.i).im
    im_xy(a * e - b * f, a * f + b * e)
    (Complex.from_real(a * e - b * f) + Complex.from_real(a * f + b * e) * Complex.i).im =
        a * f + b * e
    im_xy(c * g - d * -h, c * -h + d * g)
    (Complex.from_real(c * g - d * -h) + Complex.from_real(c * -h + d * g) * Complex.i).im =
        c * -h + d * g
    ((Complex.from_real(a * e - b * f) + Complex.from_real(a * f + b * e) * Complex.i) -
        (Complex.from_real(c * g - d * -h) + Complex.from_real(c * -h + d * g) * Complex.i)).im =
        a * f + b * e - (c * -h + d * g)
    -(c * -h + d * g) = -(c * -h) + -(d * g)
    a * f + b * e - (c * -h + d * g) = a * f + b * e + (-(c * -h) + -(d * g))
    a * f + b * e + (-(c * -h) + -(d * g)) = a * f + b * e + c * h + -(d * g)
    a * f + b * e + c * h + -(d * g) = a * f + b * e + c * h - d * g
    ((Complex.from_real(a * e - b * f) + Complex.from_real(a * f + b * e) * Complex.i) -
        (Complex.from_real(c * g - d * -h) + Complex.from_real(c * -h + d * g) * Complex.i)).im =
        a * f + b * e + c * h - d * g
    re_xy(a * e - b * f - c * g - d * h, a * f + b * e + c * h - d * g)
    (Complex.from_real(a * e - b * f - c * g - d * h) +
        Complex.from_real(a * f + b * e + c * h - d * g) * Complex.i).re =
        a * e - b * f - c * g - d * h
    im_xy(a * e - b * f - c * g - d * h, a * f + b * e + c * h - d * g)
    (Complex.from_real(a * e - b * f - c * g - d * h) +
        Complex.from_real(a * f + b * e + c * h - d * g) * Complex.i).im =
        a * f + b * e + c * h - d * g
    eq_by_components(qmul_z(Complex.from_real(a) + Complex.from_real(b) * Complex.i,
        Complex.from_real(c) + Complex.from_real(d) * Complex.i,
        Complex.from_real(e) + Complex.from_real(f) * Complex.i,
        Complex.from_real(g) + Complex.from_real(h) * Complex.i),
        Complex.from_real(a * e - b * f - c * g - d * h) +
        Complex.from_real(a * f + b * e + c * h - d * g) * Complex.i)
    qmul_z(Complex.from_real(a) + Complex.from_real(b) * Complex.i,
        Complex.from_real(c) + Complex.from_real(d) * Complex.i,
        Complex.from_real(e) + Complex.from_real(f) * Complex.i,
        Complex.from_real(g) + Complex.from_real(h) * Complex.i).re =
        (Complex.from_real(a * e - b * f - c * g - d * h) +
        Complex.from_real(a * f + b * e + c * h - d * g) * Complex.i).re
    qmul_z(Complex.from_real(a) + Complex.from_real(b) * Complex.i,
        Complex.from_real(c) + Complex.from_real(d) * Complex.i,
        Complex.from_real(e) + Complex.from_real(f) * Complex.i,
        Complex.from_real(g) + Complex.from_real(h) * Complex.i).im =
        (Complex.from_real(a * e - b * f - c * g - d * h) +
        Complex.from_real(a * f + b * e + c * h - d * g) * Complex.i).im
}

/// The second coordinate of the quaternion product of the embedded reals is
/// p3 + p4·i, where p3 and p4 are the last two product coordinates.
theorem qmul_w_coords(a: Real, b: Real, c: Real, d: Real, e: Real, f: Real, g: Real, h: Real) {
    qmul_w(Complex.from_real(a) + Complex.from_real(b) * Complex.i,
        Complex.from_real(c) + Complex.from_real(d) * Complex.i,
        Complex.from_real(e) + Complex.from_real(f) * Complex.i,
        Complex.from_real(g) + Complex.from_real(h) * Complex.i) =
        Complex.from_real(a * g - b * h + c * e + d * f) +
        Complex.from_real(a * h + b * g - c * f + d * e) * Complex.i
} by {
    qmul_w(Complex.from_real(a) + Complex.from_real(b) * Complex.i,
        Complex.from_real(c) + Complex.from_real(d) * Complex.i,
        Complex.from_real(e) + Complex.from_real(f) * Complex.i,
        Complex.from_real(g) + Complex.from_real(h) * Complex.i) =
        (Complex.from_real(a) + Complex.from_real(b) * Complex.i) *
        (Complex.from_real(g) + Complex.from_real(h) * Complex.i) +
        (Complex.from_real(c) + Complex.from_real(d) * Complex.i) *
        (Complex.from_real(e) + Complex.from_real(f) * Complex.i).conj
    z_mul_coords(a, b, g, h)
    (Complex.from_real(a) + Complex.from_real(b) * Complex.i) *
        (Complex.from_real(g) + Complex.from_real(h) * Complex.i) =
        Complex.from_real(a * g - b * h) + Complex.from_real(a * h + b * g) * Complex.i
    conj_xy(e, f)
    (Complex.from_real(e) + Complex.from_real(f) * Complex.i).conj =
        Complex.from_real(e) + Complex.from_real(-f) * Complex.i
    (Complex.from_real(c) + Complex.from_real(d) * Complex.i) *
        (Complex.from_real(e) + Complex.from_real(f) * Complex.i).conj =
        (Complex.from_real(c) + Complex.from_real(d) * Complex.i) *
        (Complex.from_real(e) + Complex.from_real(-f) * Complex.i)
    z_mul_coords(c, d, e, -f)
    (Complex.from_real(c) + Complex.from_real(d) * Complex.i) *
        (Complex.from_real(e) + Complex.from_real(-f) * Complex.i) =
        Complex.from_real(c * e - d * -f) + Complex.from_real(c * -f + d * e) * Complex.i
    qmul_w(Complex.from_real(a) + Complex.from_real(b) * Complex.i,
        Complex.from_real(c) + Complex.from_real(d) * Complex.i,
        Complex.from_real(e) + Complex.from_real(f) * Complex.i,
        Complex.from_real(g) + Complex.from_real(h) * Complex.i) =
        (Complex.from_real(a * g - b * h) + Complex.from_real(a * h + b * g) * Complex.i) +
        (Complex.from_real(c * e - d * -f) + Complex.from_real(c * -f + d * e) * Complex.i)
    re_add(Complex.from_real(a * g - b * h) + Complex.from_real(a * h + b * g) * Complex.i,
        Complex.from_real(c * e - d * -f) + Complex.from_real(c * -f + d * e) * Complex.i)
    ((Complex.from_real(a * g - b * h) + Complex.from_real(a * h + b * g) * Complex.i) +
        (Complex.from_real(c * e - d * -f) + Complex.from_real(c * -f + d * e) * Complex.i)).re =
        (Complex.from_real(a * g - b * h) + Complex.from_real(a * h + b * g) * Complex.i).re +
        (Complex.from_real(c * e - d * -f) + Complex.from_real(c * -f + d * e) * Complex.i).re
    re_xy(a * g - b * h, a * h + b * g)
    (Complex.from_real(a * g - b * h) + Complex.from_real(a * h + b * g) * Complex.i).re =
        a * g - b * h
    re_xy(c * e - d * -f, c * -f + d * e)
    (Complex.from_real(c * e - d * -f) + Complex.from_real(c * -f + d * e) * Complex.i).re =
        c * e - d * -f
    ((Complex.from_real(a * g - b * h) + Complex.from_real(a * h + b * g) * Complex.i) +
        (Complex.from_real(c * e - d * -f) + Complex.from_real(c * -f + d * e) * Complex.i)).re =
        a * g - b * h + (c * e - d * -f)
    a * g - b * h + (c * e - d * -f) = a * g - b * h + c * e + d * f
    ((Complex.from_real(a * g - b * h) + Complex.from_real(a * h + b * g) * Complex.i) +
        (Complex.from_real(c * e - d * -f) + Complex.from_real(c * -f + d * e) * Complex.i)).re =
        a * g - b * h + c * e + d * f
    im_add(Complex.from_real(a * g - b * h) + Complex.from_real(a * h + b * g) * Complex.i,
        Complex.from_real(c * e - d * -f) + Complex.from_real(c * -f + d * e) * Complex.i)
    ((Complex.from_real(a * g - b * h) + Complex.from_real(a * h + b * g) * Complex.i) +
        (Complex.from_real(c * e - d * -f) + Complex.from_real(c * -f + d * e) * Complex.i)).im =
        (Complex.from_real(a * g - b * h) + Complex.from_real(a * h + b * g) * Complex.i).im +
        (Complex.from_real(c * e - d * -f) + Complex.from_real(c * -f + d * e) * Complex.i).im
    im_xy(a * g - b * h, a * h + b * g)
    (Complex.from_real(a * g - b * h) + Complex.from_real(a * h + b * g) * Complex.i).im =
        a * h + b * g
    im_xy(c * e - d * -f, c * -f + d * e)
    (Complex.from_real(c * e - d * -f) + Complex.from_real(c * -f + d * e) * Complex.i).im =
        c * -f + d * e
    ((Complex.from_real(a * g - b * h) + Complex.from_real(a * h + b * g) * Complex.i) +
        (Complex.from_real(c * e - d * -f) + Complex.from_real(c * -f + d * e) * Complex.i)).im =
        a * h + b * g + (c * -f + d * e)
    a * h + b * g + (c * -f + d * e) = a * h + b * g - c * f + d * e
    ((Complex.from_real(a * g - b * h) + Complex.from_real(a * h + b * g) * Complex.i) +
        (Complex.from_real(c * e - d * -f) + Complex.from_real(c * -f + d * e) * Complex.i)).im =
        a * h + b * g - c * f + d * e
    re_xy(a * g - b * h + c * e + d * f, a * h + b * g - c * f + d * e)
    (Complex.from_real(a * g - b * h + c * e + d * f) +
        Complex.from_real(a * h + b * g - c * f + d * e) * Complex.i).re =
        a * g - b * h + c * e + d * f
    im_xy(a * g - b * h + c * e + d * f, a * h + b * g - c * f + d * e)
    (Complex.from_real(a * g - b * h + c * e + d * f) +
        Complex.from_real(a * h + b * g - c * f + d * e) * Complex.i).im =
        a * h + b * g - c * f + d * e
    eq_by_components(qmul_w(Complex.from_real(a) + Complex.from_real(b) * Complex.i,
        Complex.from_real(c) + Complex.from_real(d) * Complex.i,
        Complex.from_real(e) + Complex.from_real(f) * Complex.i,
        Complex.from_real(g) + Complex.from_real(h) * Complex.i),
        Complex.from_real(a * g - b * h + c * e + d * f) +
        Complex.from_real(a * h + b * g - c * f + d * e) * Complex.i)
    qmul_w(Complex.from_real(a) + Complex.from_real(b) * Complex.i,
        Complex.from_real(c) + Complex.from_real(d) * Complex.i,
        Complex.from_real(e) + Complex.from_real(f) * Complex.i,
        Complex.from_real(g) + Complex.from_real(h) * Complex.i).re =
        (Complex.from_real(a * g - b * h + c * e + d * f) +
        Complex.from_real(a * h + b * g - c * f + d * e) * Complex.i).re
    qmul_w(Complex.from_real(a) + Complex.from_real(b) * Complex.i,
        Complex.from_real(c) + Complex.from_real(d) * Complex.i,
        Complex.from_real(e) + Complex.from_real(f) * Complex.i,
        Complex.from_real(g) + Complex.from_real(h) * Complex.i).im =
        (Complex.from_real(a * g - b * h + c * e + d * f) +
        Complex.from_real(a * h + b * g - c * f + d * e) * Complex.i).im
}

/// The norm of the embedded pair (a + bi, c + di) is a² + b² + c² + d².
theorem fs4_norm(a: Real, b: Real, c: Real, d: Real) {
    fs4(a, b, c, d) =
        norm2(Complex.from_real(a) + Complex.from_real(b) * Complex.i,
            Complex.from_real(c) + Complex.from_real(d) * Complex.i)
} by {
    norm2(Complex.from_real(a) + Complex.from_real(b) * Complex.i,
        Complex.from_real(c) + Complex.from_real(d) * Complex.i) =
        (Complex.from_real(a) + Complex.from_real(b) * Complex.i).abs_squared +
        (Complex.from_real(c) + Complex.from_real(d) * Complex.i).abs_squared
    abs_sq_re_im(a, b)
    (Complex.from_real(a) + Complex.from_real(b) * Complex.i).abs_squared = a * a + b * b
    abs_sq_re_im(c, d)
    (Complex.from_real(c) + Complex.from_real(d) * Complex.i).abs_squared = c * c + d * d
    norm2(Complex.from_real(a) + Complex.from_real(b) * Complex.i,
        Complex.from_real(c) + Complex.from_real(d) * Complex.i) =
        (a * a + b * b) + (c * c + d * d)
    a * a + b * b + c * c + d * d = (a * a + b * b) + (c * c + d * d)
    norm2(Complex.from_real(a) + Complex.from_real(b) * Complex.i,
        Complex.from_real(c) + Complex.from_real(d) * Complex.i) =
        a * a + b * b + c * c + d * d
    fs4(a, b, c, d) = a * a + b * b + c * c + d * d
    fs4(a, b, c, d) =
        norm2(Complex.from_real(a) + Complex.from_real(b) * Complex.i,
            Complex.from_real(c) + Complex.from_real(d) * Complex.i)
}

/// Euler's four-squares identity: the norm of the quaternion product is the
/// product of the norms, over the reals:
/// (a²+b²+c²+d²)(e²+f²+g²+h²) = p1² + p2² + p3² + p4² with the p_i the
/// coordinates of the quaternion product.
theorem four_squares_identity(a: Real, b: Real, c: Real, d: Real, e: Real, f: Real, g: Real, h: Real) {
    fs4(a, b, c, d) * fs4(e, f, g, h) =
        fs4(qp1(a, b, c, d, e, f, g, h), qp2(a, b, c, d, e, f, g, h),
            qp3(a, b, c, d, e, f, g, h), qp4(a, b, c, d, e, f, g, h))
} by {
    fs4_norm(a, b, c, d)
    fs4(a, b, c, d) =
        norm2(Complex.from_real(a) + Complex.from_real(b) * Complex.i,
            Complex.from_real(c) + Complex.from_real(d) * Complex.i)
    fs4_norm(e, f, g, h)
    fs4(e, f, g, h) =
        norm2(Complex.from_real(e) + Complex.from_real(f) * Complex.i,
            Complex.from_real(g) + Complex.from_real(h) * Complex.i)
    fs4(a, b, c, d) * fs4(e, f, g, h) =
        norm2(Complex.from_real(a) + Complex.from_real(b) * Complex.i,
            Complex.from_real(c) + Complex.from_real(d) * Complex.i) *
        norm2(Complex.from_real(e) + Complex.from_real(f) * Complex.i,
            Complex.from_real(g) + Complex.from_real(h) * Complex.i)
    norm2_mul(Complex.from_real(a) + Complex.from_real(b) * Complex.i,
        Complex.from_real(c) + Complex.from_real(d) * Complex.i,
        Complex.from_real(e) + Complex.from_real(f) * Complex.i,
        Complex.from_real(g) + Complex.from_real(h) * Complex.i)
    norm2(Complex.from_real(a) + Complex.from_real(b) * Complex.i,
        Complex.from_real(c) + Complex.from_real(d) * Complex.i) *
        norm2(Complex.from_real(e) + Complex.from_real(f) * Complex.i,
            Complex.from_real(g) + Complex.from_real(h) * Complex.i) =
        norm2(qmul_z(Complex.from_real(a) + Complex.from_real(b) * Complex.i,
                Complex.from_real(c) + Complex.from_real(d) * Complex.i,
                Complex.from_real(e) + Complex.from_real(f) * Complex.i,
                Complex.from_real(g) + Complex.from_real(h) * Complex.i),
            qmul_w(Complex.from_real(a) + Complex.from_real(b) * Complex.i,
                Complex.from_real(c) + Complex.from_real(d) * Complex.i,
                Complex.from_real(e) + Complex.from_real(f) * Complex.i,
                Complex.from_real(g) + Complex.from_real(h) * Complex.i))
    qmul_z_coords(a, b, c, d, e, f, g, h)
    qmul_z(Complex.from_real(a) + Complex.from_real(b) * Complex.i,
        Complex.from_real(c) + Complex.from_real(d) * Complex.i,
        Complex.from_real(e) + Complex.from_real(f) * Complex.i,
        Complex.from_real(g) + Complex.from_real(h) * Complex.i) =
        Complex.from_real(a * e - b * f - c * g - d * h) +
        Complex.from_real(a * f + b * e + c * h - d * g) * Complex.i
    qmul_w_coords(a, b, c, d, e, f, g, h)
    qmul_w(Complex.from_real(a) + Complex.from_real(b) * Complex.i,
        Complex.from_real(c) + Complex.from_real(d) * Complex.i,
        Complex.from_real(e) + Complex.from_real(f) * Complex.i,
        Complex.from_real(g) + Complex.from_real(h) * Complex.i) =
        Complex.from_real(a * g - b * h + c * e + d * f) +
        Complex.from_real(a * h + b * g - c * f + d * e) * Complex.i
    norm2(qmul_z(Complex.from_real(a) + Complex.from_real(b) * Complex.i,
            Complex.from_real(c) + Complex.from_real(d) * Complex.i,
            Complex.from_real(e) + Complex.from_real(f) * Complex.i,
            Complex.from_real(g) + Complex.from_real(h) * Complex.i),
        qmul_w(Complex.from_real(a) + Complex.from_real(b) * Complex.i,
            Complex.from_real(c) + Complex.from_real(d) * Complex.i,
            Complex.from_real(e) + Complex.from_real(f) * Complex.i,
            Complex.from_real(g) + Complex.from_real(h) * Complex.i)) =
        norm2(Complex.from_real(a * e - b * f - c * g - d * h) +
            Complex.from_real(a * f + b * e + c * h - d * g) * Complex.i,
            Complex.from_real(a * g - b * h + c * e + d * f) +
            Complex.from_real(a * h + b * g - c * f + d * e) * Complex.i)
    abs_sq_re_im(a * e - b * f - c * g - d * h, a * f + b * e + c * h - d * g)
    (Complex.from_real(a * e - b * f - c * g - d * h) +
        Complex.from_real(a * f + b * e + c * h - d * g) * Complex.i).abs_squared =
        (a * e - b * f - c * g - d * h) * (a * e - b * f - c * g - d * h) +
        (a * f + b * e + c * h - d * g) * (a * f + b * e + c * h - d * g)
    abs_sq_re_im(a * g - b * h + c * e + d * f, a * h + b * g - c * f + d * e)
    (Complex.from_real(a * g - b * h + c * e + d * f) +
        Complex.from_real(a * h + b * g - c * f + d * e) * Complex.i).abs_squared =
        (a * g - b * h + c * e + d * f) * (a * g - b * h + c * e + d * f) +
        (a * h + b * g - c * f + d * e) * (a * h + b * g - c * f + d * e)
    norm2(Complex.from_real(a * e - b * f - c * g - d * h) +
        Complex.from_real(a * f + b * e + c * h - d * g) * Complex.i,
        Complex.from_real(a * g - b * h + c * e + d * f) +
        Complex.from_real(a * h + b * g - c * f + d * e) * Complex.i) =
        (a * e - b * f - c * g - d * h) * (a * e - b * f - c * g - d * h) +
        (a * f + b * e + c * h - d * g) * (a * f + b * e + c * h - d * g) +
        (a * g - b * h + c * e + d * f) * (a * g - b * h + c * e + d * f) +
        (a * h + b * g - c * f + d * e) * (a * h + b * g - c * f + d * e)
    qp1(a, b, c, d, e, f, g, h) = a * e - b * f - c * g - d * h
    qp2(a, b, c, d, e, f, g, h) = a * f + b * e + c * h - d * g
    qp3(a, b, c, d, e, f, g, h) = a * g - b * h + c * e + d * f
    qp4(a, b, c, d, e, f, g, h) = a * h + b * g - c * f + d * e
    fs4(qp1(a, b, c, d, e, f, g, h), qp2(a, b, c, d, e, f, g, h),
        qp3(a, b, c, d, e, f, g, h), qp4(a, b, c, d, e, f, g, h)) =
        qp1(a, b, c, d, e, f, g, h) * qp1(a, b, c, d, e, f, g, h) +
        qp2(a, b, c, d, e, f, g, h) * qp2(a, b, c, d, e, f, g, h) +
        qp3(a, b, c, d, e, f, g, h) * qp3(a, b, c, d, e, f, g, h) +
        qp4(a, b, c, d, e, f, g, h) * qp4(a, b, c, d, e, f, g, h)
    fs4(a, b, c, d) * fs4(e, f, g, h) =
        fs4(qp1(a, b, c, d, e, f, g, h), qp2(a, b, c, d, e, f, g, h),
            qp3(a, b, c, d, e, f, g, h), qp4(a, b, c, d, e, f, g, h))
}
