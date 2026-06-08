from list import List
from pair import Pair
from number_theory.congruence import Nat, mod_congr_mod_self, mod_lt, congr_mod_symm
from number_theory.pairwise_coprime import pairwise_coprime
from number_theory.crt_list import satisfies_all, system_moduli, system_modulus,
    every_modulus_positive, every_modulus_positive_imp_modulus_nonzero,
    satisfies_all_descend, nat_crt_list,
    satisfies_all_unique_mod_system_modulus, congr_mod_below_system_modulus_eq
numerals Nat

/// A solution remains a solution after reducing it modulo the combined system modulus.
theorem normalized_representative_satisfies_all(system: List[Pair[Nat, Nat]], c: Nat) {
    every_modulus_positive(system) and satisfies_all(c, system)
        implies satisfies_all(c.mod(system_modulus(system)), system)
} by {
    if every_modulus_positive(system) and satisfies_all(c, system) {
        let m: Nat = system_modulus(system)
        mod_congr_mod_self(c, m)
        c.mod(m).congr_mod(c, m)
        satisfies_all_descend(c.mod(m), c, system)
        satisfies_all(c.mod(m), system)
        satisfies_all(c.mod(system_modulus(system)), system)
    }
}

/// The reduced representative is strictly below the combined modulus for a positive system.
theorem normalized_representative_lt_system_modulus(system: List[Pair[Nat, Nat]], c: Nat) {
    every_modulus_positive(system) implies c.mod(system_modulus(system)) < system_modulus(system)
} by {
    if every_modulus_positive(system) {
        every_modulus_positive_imp_modulus_nonzero(system)
        let m: Nat = system_modulus(system)
        m != Nat.0
        mod_lt(c, m)
        c.mod(m) < m
        c.mod(system_modulus(system)) < system_modulus(system)
    }
}

/// A positive pairwise-coprime system has a normalized simultaneous solution.
theorem normalized_crt_solution_exists(system: List[Pair[Nat, Nat]]) {
    pairwise_coprime(system_moduli(system)) and every_modulus_positive(system)
        implies exists(c: Nat) {
            satisfies_all(c, system) and c < system_modulus(system)
        }
} by {
    if pairwise_coprime(system_moduli(system)) and every_modulus_positive(system) {
        nat_crt_list(system)
        let c0: Nat satisfy { satisfies_all(c0, system) }
        normalized_representative_satisfies_all(system, c0)
        normalized_representative_lt_system_modulus(system, c0)
        let c: Nat = c0.mod(system_modulus(system))
        satisfies_all(c, system)
        c < system_modulus(system)
        exists(result: Nat) {
            satisfies_all(result, system) and result < system_modulus(system)
        }
    }
}

/// There is a value satisfying the canonical-solution specification.
theorem crt_canonical_solution_choice_exists(system: List[Pair[Nat, Nat]]) {
    exists(result: Nat) {
        pairwise_coprime(system_moduli(system)) and every_modulus_positive(system) implies
            satisfies_all(result, system) and result < system_modulus(system)
    }
} by {
    if pairwise_coprime(system_moduli(system)) and every_modulus_positive(system) {
        normalized_crt_solution_exists(system)
        let c: Nat satisfy {
            satisfies_all(c, system) and c < system_modulus(system)
        }
        exists(result: Nat) {
            pairwise_coprime(system_moduli(system)) and every_modulus_positive(system) implies
                satisfies_all(result, system) and result < system_modulus(system)
        }
    } else {
        exists(result: Nat) {
            pairwise_coprime(system_moduli(system)) and every_modulus_positive(system) implies
                satisfies_all(result, system) and result < system_modulus(system)
        }
    }
}

/// The canonical normalized solution of a positive pairwise-coprime CRT system.
let crt_canonical_solution(system: List[Pair[Nat, Nat]]) -> result: Nat satisfy {
    pairwise_coprime(system_moduli(system)) and every_modulus_positive(system) implies
        satisfies_all(result, system) and result < system_modulus(system)
} by {
    crt_canonical_solution_choice_exists(system)
}

/// The canonical CRT solution satisfies every congruence in the system.
theorem crt_canonical_satisfies_all(system: List[Pair[Nat, Nat]]) {
    pairwise_coprime(system_moduli(system)) and every_modulus_positive(system)
        implies satisfies_all(crt_canonical_solution(system), system)
} by {
    if pairwise_coprime(system_moduli(system)) and every_modulus_positive(system) {
        crt_canonical_solution(system) = crt_canonical_solution(system)
        satisfies_all(crt_canonical_solution(system), system)
    }
}

/// The canonical CRT solution is normalized below the combined system modulus.
theorem crt_canonical_lt_system_modulus(system: List[Pair[Nat, Nat]]) {
    pairwise_coprime(system_moduli(system)) and every_modulus_positive(system)
        implies crt_canonical_solution(system) < system_modulus(system)
} by {
    if pairwise_coprime(system_moduli(system)) and every_modulus_positive(system) {
        crt_canonical_solution(system) = crt_canonical_solution(system)
        crt_canonical_solution(system) < system_modulus(system)
    }
}

/// The canonical CRT solution is congruent modulo the combined modulus to every solution.
theorem crt_canonical_congruent_to_solution(system: List[Pair[Nat, Nat]], c: Nat) {
    pairwise_coprime(system_moduli(system))
        and every_modulus_positive(system)
        and satisfies_all(c, system)
        implies crt_canonical_solution(system).congr_mod(c, system_modulus(system))
} by {
    if pairwise_coprime(system_moduli(system))
        and every_modulus_positive(system)
        and satisfies_all(c, system) {
        crt_canonical_satisfies_all(system)
        satisfies_all_unique_mod_system_modulus(system, crt_canonical_solution(system), c)
        crt_canonical_solution(system).congr_mod(c, system_modulus(system))
    }
}

/// Every solution is congruent to the canonical CRT solution modulo the combined modulus.
theorem solution_congruent_to_crt_canonical(system: List[Pair[Nat, Nat]], c: Nat) {
    pairwise_coprime(system_moduli(system))
        and every_modulus_positive(system)
        and satisfies_all(c, system)
        implies c.congr_mod(crt_canonical_solution(system), system_modulus(system))
} by {
    if pairwise_coprime(system_moduli(system))
        and every_modulus_positive(system)
        and satisfies_all(c, system) {
        crt_canonical_congruent_to_solution(system, c)
        congr_mod_symm(crt_canonical_solution(system), c, system_modulus(system))
        c.congr_mod(crt_canonical_solution(system), system_modulus(system))
    }
}

/// Any normalized simultaneous solution is the canonical CRT solution.
theorem normalized_solution_eq_crt_canonical(system: List[Pair[Nat, Nat]], c: Nat) {
    pairwise_coprime(system_moduli(system))
        and every_modulus_positive(system)
        and satisfies_all(c, system)
        and c < system_modulus(system)
        implies c = crt_canonical_solution(system)
} by {
    if pairwise_coprime(system_moduli(system))
        and every_modulus_positive(system)
        and satisfies_all(c, system)
        and c < system_modulus(system) {
        solution_congruent_to_crt_canonical(system, c)
        c.congr_mod(crt_canonical_solution(system), system_modulus(system))
        crt_canonical_lt_system_modulus(system)
        congr_mod_below_system_modulus_eq(system, c, crt_canonical_solution(system))
        c = crt_canonical_solution(system)
    }
}

/// The canonical CRT solution is uniquely characterized by satisfaction and normalization.
theorem crt_canonical_solution_unique(system: List[Pair[Nat, Nat]], c: Nat) {
    pairwise_coprime(system_moduli(system))
        and every_modulus_positive(system)
        implies ((satisfies_all(c, system) and c < system_modulus(system)) =
            (c = crt_canonical_solution(system)))
} by {
    if pairwise_coprime(system_moduli(system)) and every_modulus_positive(system) {
        if satisfies_all(c, system) and c < system_modulus(system) {
            normalized_solution_eq_crt_canonical(system, c)
            c = crt_canonical_solution(system)
        }
        if c = crt_canonical_solution(system) {
            crt_canonical_satisfies_all(system)
            crt_canonical_lt_system_modulus(system)
            satisfies_all(c, system)
            c < system_modulus(system)
            satisfies_all(c, system) and c < system_modulus(system)
        }
        (satisfies_all(c, system) and c < system_modulus(system)) =
            (c = crt_canonical_solution(system))
    }
}
