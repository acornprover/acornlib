/// Polynomial number theory: the rational root theorem, the root-count
/// bound for polynomials over a field, the arithmetic of `X^2 + 1` (no
/// rational root, complex roots `+-i`), and the analytic root theorems for
/// real polynomials (continuity of evaluation and the sign-change root
/// theorem, which is the intermediate value theorem application behind
/// "every odd-degree real polynomial has a real root").
///
/// The rational root theorem (part 1) is stated in full for a general
/// integer polynomial and proved by invoking the library's coefficient-level
/// proof in `polynomial_rational_root_bridge.ac` (`rat_root_numerator_divides`
/// and `rat_root_denominator_divides`), after transporting the root condition
/// from global polynomial evaluation to bounded coefficient evaluation.  The
/// quadratic case is spelled out with explicit leading/constant coefficients.
///
/// The root-count bound (part 2) is a restatement of
/// `polynomial_root_list_length_lt_support_bound` from `src/polynomial/`:
/// "a nonzero polynomial over a field has fewer than `n` distinct roots when
/// its support is below `n`", which is the classical statement that a degree
/// `n` polynomial has at most `n` roots (the degree is the least such
/// bound, so `roots.length < n` is the sharp form).
///
/// Part 3 verifies `X^2 + 1`: over the rationals it has no root (a square is
/// nonnegative and one is positive), while over the complex numbers both
/// `i` and `-i` are roots, using the quadratic-formula evaluation machinery
/// of `src/complex/fundamental_theorem_algebra.ac`.
///
/// Part 4 proves that evaluation at a real polynomial is a continuous
/// function (by induction on the Horner bound), and that a continuous real
/// function taking both signs has a real zero (Bolzano's theorem, i.e. the
/// intermediate value theorem at target zero).  The classical consequence --
/// every real polynomial of odd degree with nonzero leading coefficient has
/// a real root -- is stated below in comments: it additionally needs the
/// leading-term dominance lemma (the value of the polynomial is eventually
/// positive and eventually negative), which requires real limit machinery
/// not yet in the library.

from nat import Nat, alt_induction, lt_suc, lt_trans
from int import Int
from rat import Rat, pos_imp_zero_lt, neg_imp_lt_zero, one_is_pos, lt_add_pos,
    mul_nonnegative, single_trichotomy, not_lt_self
from list import List
from semiring import Semiring
from algebra.field.field import Field
from algebra.ring.ring import mul_neg_neg
from algebra.add_ordered_group import negative_of_negative, neg_lt_neg
from order import lt_imp_lte, lte_refl, lte_lt_trans
from data.int.int_coprime import is_coprime
from data.basic.functions import function_extensionality, identity_fn
from data.basic.function_algebra import pointwise_mul
from polynomial import Polynomial, polynomial_eval, polynomial_constant, polynomial_monomial,
    polynomial_add_coeff, polynomial_monomial_coeff_of_ne, polynomial_monomial_coeff_self,
    polynomial_constant_coeff_zero, polynomial_constant_coeff_of_ne_zero,
    polynomial_constant_support_bounded_by_one, polynomial_add_support_bounded_by,
    polynomial_support_bounded_by, polynomial_support_bounded_by_monotone,
    polynomial_eval_bound_eq_coeff_eval, polynomial_eval_eq_eval_bound_of_support_bounded,
    polynomial_eval_bound, polynomial_eval_add, polynomial_eval_constant,
    polynomial_support_bound_exists,
    polynomial_roots_on_list, polynomial_root_list_length_lt_support_bound,
    coeff_eval, coeff_tail, coeff_eval_zero_index
from polynomial_int_roots import rationalise, rationalise_support_bounded_by, rationalise_coeff
from polynomial_rational_root_bridge import rat_coeffs, rat_coeffs_apply,
    rat_root_numerator_divides, rat_root_denominator_divides
from complex import Complex, i_squared_eq_neg_one, im_i, neg_im, mul_assoc,
    add_neg_cancel, quadratic_polynomial, polynomial_eval_monomial_one,
    polynomial_eval_monomial_two, polynomial_monomial_support_bounded_by_suc
from real import Real, two, two_nonzero, mul_neg_one_left, lte_or_gte, neg_zero,
    neg_neg, continuous, const_add_left, const_mul_left, constant_function_is_continuous,
    identity_function_is_continuous, continuous_pointwise_mul, continuous_const_add_left,
    continuous_const_mul_left, intermediate_value_closed_interval
from order_set import closed_interval_set
from algebra.add_group import inverse_left

numerals Nat
numerals Int
numerals Rat

// ---------------------------------------------------------------------------
// 1. The rational root theorem
// ---------------------------------------------------------------------------

/// The quadratic `a * X^2 + b * X + c` with integer coefficients.
define int_quadratic_polynomial(a: Int, b: Int, c: Int) -> Polynomial[Int] {
    polynomial_constant(c) + polynomial_monomial(Nat.1, b) + polynomial_monomial(Nat.2, a)
}

/// The constant coefficient of an integer quadratic is its constant term.
theorem int_quadratic_coeff_zero(a: Int, b: Int, c: Int) {
    int_quadratic_polynomial(a, b, c).coeff(Nat.0) = c
} by {
    polynomial_add_coeff(polynomial_constant(c) + polynomial_monomial(Nat.1, b),
        polynomial_monomial(Nat.2, a), Nat.0)
    int_quadratic_polynomial(a, b, c).coeff(Nat.0) =
        (polynomial_constant(c) + polynomial_monomial(Nat.1, b)).coeff(Nat.0) +
        polynomial_monomial(Nat.2, a).coeff(Nat.0)
    polynomial_add_coeff(polynomial_constant(c), polynomial_monomial(Nat.1, b), Nat.0)
    (polynomial_constant(c) + polynomial_monomial(Nat.1, b)).coeff(Nat.0) =
        polynomial_constant(c).coeff(Nat.0) + polynomial_monomial(Nat.1, b).coeff(Nat.0)
    polynomial_constant_coeff_zero(c)
    polynomial_constant(c).coeff(Nat.0) = c
    Nat.0 != Nat.1
    polynomial_monomial_coeff_of_ne[Int](Nat.1, b, Nat.0)
    polynomial_monomial(Nat.1, b).coeff(Nat.0) = Int.0
    Nat.0 != Nat.2
    polynomial_monomial_coeff_of_ne[Int](Nat.2, a, Nat.0)
    polynomial_monomial(Nat.2, a).coeff(Nat.0) = Int.0
    int_quadratic_polynomial(a, b, c).coeff(Nat.0) = c + Int.0 + Int.0
    c + Int.0 + Int.0 = c
    int_quadratic_polynomial(a, b, c).coeff(Nat.0) = c
}

/// The quadratic coefficient of an integer quadratic is its leading term.
theorem int_quadratic_coeff_two(a: Int, b: Int, c: Int) {
    int_quadratic_polynomial(a, b, c).coeff(Nat.2) = a
} by {
    polynomial_add_coeff(polynomial_constant(c) + polynomial_monomial(Nat.1, b),
        polynomial_monomial(Nat.2, a), Nat.2)
    int_quadratic_polynomial(a, b, c).coeff(Nat.2) =
        (polynomial_constant(c) + polynomial_monomial(Nat.1, b)).coeff(Nat.2) +
        polynomial_monomial(Nat.2, a).coeff(Nat.2)
    polynomial_add_coeff(polynomial_constant(c), polynomial_monomial(Nat.1, b), Nat.2)
    (polynomial_constant(c) + polynomial_monomial(Nat.1, b)).coeff(Nat.2) =
        polynomial_constant(c).coeff(Nat.2) + polynomial_monomial(Nat.1, b).coeff(Nat.2)
    Nat.2 != Nat.0
    polynomial_constant_coeff_of_ne_zero(c, Nat.2)
    polynomial_constant(c).coeff(Nat.2) = Int.0
    Nat.1 != Nat.2
    polynomial_monomial_coeff_of_ne[Int](Nat.1, b, Nat.2)
    polynomial_monomial(Nat.1, b).coeff(Nat.2) = Int.0
    polynomial_monomial_coeff_self(Nat.2, a)
    polynomial_monomial(Nat.2, a).coeff(Nat.2) = a
    int_quadratic_polynomial(a, b, c).coeff(Nat.2) = Int.0 + Int.0 + a
    Int.0 + Int.0 + a = a
    int_quadratic_polynomial(a, b, c).coeff(Nat.2) = a
}

/// Two is at most three.
lemma nat_two_le_three {
    Nat.1.suc <= Nat.2.suc
} by {
    lt_suc(Nat.1)
    Nat.1 < Nat.1.suc
    Nat.1.suc = Nat.2
    Nat.1 < Nat.2
    lt_suc(Nat.2)
    Nat.2 < Nat.2.suc
    lt_trans(Nat.1, Nat.2, Nat.2.suc)
    Nat.1 < Nat.2.suc
    lt_imp_lte[Nat](Nat.1, Nat.2.suc)
    Nat.1 <= Nat.2.suc
    Nat.1.suc = Nat.2
    Nat.1.suc <= Nat.2.suc
}

/// One is at most three.
lemma nat_one_le_three {
    Nat.0.suc <= Nat.2.suc
} by {
    lt_suc(Nat.1)
    Nat.1 < Nat.1.suc
    Nat.1.suc = Nat.2
    Nat.1 < Nat.2
    lt_suc(Nat.2)
    Nat.2 < Nat.2.suc
    lt_trans(Nat.1, Nat.2, Nat.2.suc)
    Nat.1 < Nat.2.suc
    lt_imp_lte[Nat](Nat.1, Nat.2.suc)
    Nat.1 <= Nat.2.suc
    Nat.0.suc = Nat.1
    Nat.0.suc <= Nat.2.suc
}

/// An integer quadratic is supported below three coefficient slots.
theorem int_quadratic_support_bounded_by_three(a: Int, b: Int, c: Int) {
    polynomial_support_bounded_by(int_quadratic_polynomial(a, b, c), Nat.0.suc.suc.suc)
} by {
    polynomial_constant_support_bounded_by_one(c)
    polynomial_support_bounded_by(polynomial_constant(c), Nat.0.suc)
    polynomial_monomial_support_bounded_by_suc[Int](Nat.1, b)
    polynomial_support_bounded_by(polynomial_monomial(Nat.1, b), Nat.1.suc)
    nat_two_le_three
    polynomial_support_bounded_by_monotone(polynomial_monomial(Nat.1, b), Nat.1.suc, Nat.2.suc)
    polynomial_support_bounded_by(polynomial_monomial(Nat.1, b), Nat.2.suc)
    nat_one_le_three
    polynomial_support_bounded_by_monotone(polynomial_constant(c), Nat.0.suc, Nat.2.suc)
    polynomial_support_bounded_by(polynomial_constant(c), Nat.2.suc)
    polynomial_add_support_bounded_by(polynomial_constant(c), polynomial_monomial(Nat.1, b),
        Nat.2.suc)
    polynomial_support_bounded_by(polynomial_constant(c) + polynomial_monomial(Nat.1, b),
        Nat.2.suc)
    polynomial_monomial_support_bounded_by_suc[Int](Nat.2, a)
    polynomial_support_bounded_by(polynomial_monomial(Nat.2, a), Nat.2.suc)
    polynomial_add_support_bounded_by(polynomial_constant(c) + polynomial_monomial(Nat.1, b),
        polynomial_monomial(Nat.2, a), Nat.2.suc)
    polynomial_support_bounded_by(
        (polynomial_constant(c) + polynomial_monomial(Nat.1, b)) + polynomial_monomial(Nat.2, a),
        Nat.2.suc)
    polynomial_support_bounded_by(int_quadratic_polynomial(a, b, c), Nat.2.suc)
    polynomial_support_bounded_by(int_quadratic_polynomial(a, b, c), Nat.0.suc.suc.suc)
}

/// The rational root theorem, first half: the numerator of a rational root
/// in lowest terms divides the constant coefficient.
///
/// `p` is an integer polynomial supported below `n + 1` (so `p.coeff(n)` is
/// the leading coefficient), `a / b` is the root in lowest terms
/// (`b != 0` and `is_coprime(a, b)`), and the root condition is evaluated in
/// the rationals after embedding the polynomial.  The conclusion is
/// `a | p.coeff(0)`.
theorem polynomial_rational_root_numerator_divides(p: Polynomial[Int], a: Int, b: Int, n: Nat) {
    polynomial_support_bounded_by(p, n.suc) and b != Int.0 and is_coprime(a, b) and
    polynomial_eval(rationalise(p), Rat.from_int(a) / Rat.from_int(b)) = Rat.0
    implies a.divides(p.coeff(Nat.0))
} by {
    if polynomial_support_bounded_by(p, n.suc) and b != Int.0 and is_coprime(a, b) and
        polynomial_eval(rationalise(p), Rat.from_int(a) / Rat.from_int(b)) = Rat.0 {
        rationalise_support_bounded_by(p, n.suc)
        polynomial_support_bounded_by(rationalise(p), n.suc)
        polynomial_eval_eq_eval_bound_of_support_bounded(rationalise(p),
            Rat.from_int(a) / Rat.from_int(b), n.suc)
        polynomial_eval(rationalise(p), Rat.from_int(a) / Rat.from_int(b)) =
            polynomial_eval_bound(rationalise(p), Rat.from_int(a) / Rat.from_int(b), n.suc)
        polynomial_eval_bound(rationalise(p), Rat.from_int(a) / Rat.from_int(b), n.suc) = Rat.0
        polynomial_eval_bound_eq_coeff_eval(rationalise(p),
            Rat.from_int(a) / Rat.from_int(b), n.suc)
        polynomial_eval_bound(rationalise(p), Rat.from_int(a) / Rat.from_int(b), n.suc) =
            coeff_eval(rationalise(p).coeff, Rat.from_int(a) / Rat.from_int(b), n.suc)
        coeff_eval(rationalise(p).coeff, Rat.from_int(a) / Rat.from_int(b), n.suc) = Rat.0
        forall(i: Nat) {
            rationalise_coeff(p, i)
            rationalise(p).coeff(i) = Rat.from_int(p.coeff(i))
            rat_coeffs_apply(p.coeff, i)
            rat_coeffs(p.coeff)(i) = Rat.from_int(p.coeff(i))
            rationalise(p).coeff(i) = rat_coeffs(p.coeff)(i)
        }
        function_extensionality[Nat, Rat](rationalise(p).coeff, rat_coeffs(p.coeff))
        rationalise(p).coeff = rat_coeffs(p.coeff)
        coeff_eval(rationalise(p).coeff, Rat.from_int(a) / Rat.from_int(b), n.suc) =
            coeff_eval(rat_coeffs(p.coeff), Rat.from_int(a) / Rat.from_int(b), n.suc)
        coeff_eval(rat_coeffs(p.coeff), Rat.from_int(a) / Rat.from_int(b), n.suc) = Rat.0
        rat_root_numerator_divides(p.coeff, a, b, n)
        a.divides(p.coeff(Nat.0))
    }
}

/// The rational root theorem, second half: the denominator of a rational
/// root in lowest terms divides the leading coefficient `p.coeff(n)`.
theorem polynomial_rational_root_denominator_divides(p: Polynomial[Int], a: Int, b: Int, n: Nat) {
    polynomial_support_bounded_by(p, n.suc) and b != Int.0 and is_coprime(a, b) and
    polynomial_eval(rationalise(p), Rat.from_int(a) / Rat.from_int(b)) = Rat.0
    implies b.divides(p.coeff(n))
} by {
    if polynomial_support_bounded_by(p, n.suc) and b != Int.0 and is_coprime(a, b) and
        polynomial_eval(rationalise(p), Rat.from_int(a) / Rat.from_int(b)) = Rat.0 {
        rationalise_support_bounded_by(p, n.suc)
        polynomial_support_bounded_by(rationalise(p), n.suc)
        polynomial_eval_eq_eval_bound_of_support_bounded(rationalise(p),
            Rat.from_int(a) / Rat.from_int(b), n.suc)
        polynomial_eval(rationalise(p), Rat.from_int(a) / Rat.from_int(b)) =
            polynomial_eval_bound(rationalise(p), Rat.from_int(a) / Rat.from_int(b), n.suc)
        polynomial_eval_bound(rationalise(p), Rat.from_int(a) / Rat.from_int(b), n.suc) = Rat.0
        polynomial_eval_bound_eq_coeff_eval(rationalise(p),
            Rat.from_int(a) / Rat.from_int(b), n.suc)
        polynomial_eval_bound(rationalise(p), Rat.from_int(a) / Rat.from_int(b), n.suc) =
            coeff_eval(rationalise(p).coeff, Rat.from_int(a) / Rat.from_int(b), n.suc)
        coeff_eval(rationalise(p).coeff, Rat.from_int(a) / Rat.from_int(b), n.suc) = Rat.0
        forall(i: Nat) {
            rationalise_coeff(p, i)
            rationalise(p).coeff(i) = Rat.from_int(p.coeff(i))
            rat_coeffs_apply(p.coeff, i)
            rat_coeffs(p.coeff)(i) = Rat.from_int(p.coeff(i))
            rationalise(p).coeff(i) = rat_coeffs(p.coeff)(i)
        }
        function_extensionality[Nat, Rat](rationalise(p).coeff, rat_coeffs(p.coeff))
        rationalise(p).coeff = rat_coeffs(p.coeff)
        coeff_eval(rationalise(p).coeff, Rat.from_int(a) / Rat.from_int(b), n.suc) =
            coeff_eval(rat_coeffs(p.coeff), Rat.from_int(a) / Rat.from_int(b), n.suc)
        coeff_eval(rat_coeffs(p.coeff), Rat.from_int(a) / Rat.from_int(b), n.suc) = Rat.0
        rat_root_denominator_divides(p.coeff, a, b, n)
        b.divides(p.coeff(n))
    }
}

/// The rational root theorem for quadratics: the numerator of a rational
/// root of `a X^2 + b X + c` in lowest terms divides the constant term `c`.
theorem quadratic_rational_root_numerator_divides(a: Int, b: Int, c: Int, p: Int, q: Int) {
    q != Int.0 and is_coprime(p, q) and
    polynomial_eval(rationalise(int_quadratic_polynomial(a, b, c)),
        Rat.from_int(p) / Rat.from_int(q)) = Rat.0
    implies p.divides(c)
} by {
    if q != Int.0 and is_coprime(p, q) and
        polynomial_eval(rationalise(int_quadratic_polynomial(a, b, c)),
            Rat.from_int(p) / Rat.from_int(q)) = Rat.0 {
        int_quadratic_support_bounded_by_three(a, b, c)
        polynomial_support_bounded_by(int_quadratic_polynomial(a, b, c), Nat.0.suc.suc.suc)
        polynomial_rational_root_numerator_divides(int_quadratic_polynomial(a, b, c), p, q, Nat.2)
        p.divides(int_quadratic_polynomial(a, b, c).coeff(Nat.0))
        int_quadratic_coeff_zero(a, b, c)
        int_quadratic_polynomial(a, b, c).coeff(Nat.0) = c
        p.divides(c)
    }
}

/// The rational root theorem for quadratics: the denominator of a rational
/// root of `a X^2 + b X + c` in lowest terms divides the leading term `a`.
theorem quadratic_rational_root_denominator_divides(a: Int, b: Int, c: Int, p: Int, q: Int) {
    q != Int.0 and is_coprime(p, q) and
    polynomial_eval(rationalise(int_quadratic_polynomial(a, b, c)),
        Rat.from_int(p) / Rat.from_int(q)) = Rat.0
    implies q.divides(a)
} by {
    if q != Int.0 and is_coprime(p, q) and
        polynomial_eval(rationalise(int_quadratic_polynomial(a, b, c)),
            Rat.from_int(p) / Rat.from_int(q)) = Rat.0 {
        int_quadratic_support_bounded_by_three(a, b, c)
        polynomial_support_bounded_by(int_quadratic_polynomial(a, b, c), Nat.0.suc.suc.suc)
        polynomial_rational_root_denominator_divides(int_quadratic_polynomial(a, b, c), p, q, Nat.2)
        q.divides(int_quadratic_polynomial(a, b, c).coeff(Nat.2))
        int_quadratic_coeff_two(a, b, c)
        int_quadratic_polynomial(a, b, c).coeff(Nat.2) = a
        q.divides(a)
    }
}

// ---------------------------------------------------------------------------
// 2. A polynomial over a field has at most its degree many roots
// ---------------------------------------------------------------------------

/// A nonzero polynomial over a field whose support lies below `n` has fewer
/// than `n` distinct roots in any explicitly listed collection.
///
/// This is the classical "a polynomial of degree `n` over a field has at
/// most `n` roots": the degree is the least natural `d` with the support
/// below `d + 1`, so a root list of distinct roots has length at most the
/// degree.  The statement is proved in `src/polynomial/root_bound.ac`
/// (`polynomial_root_list_length_lt_support_bound`); it is restated here as
/// the number-theoretic counting principle.
theorem polynomial_root_count_lt_support_bound[F: Field](
    p: Polynomial[F], roots: List[F], n: Nat
) {
    polynomial_support_bounded_by(p, n) and p != Polynomial[F].zero and roots.is_unique and
    polynomial_roots_on_list(p, roots)
    implies roots.length < n
} by {
    polynomial_root_list_length_lt_support_bound(p, roots, n)
    roots.length < n
}

// ---------------------------------------------------------------------------
// 3. X^2 + 1: no rational roots, complex roots +-i
// ---------------------------------------------------------------------------

/// The polynomial `X^2 + 1` with rational coefficients.
let rat_x_squared_plus_one: Polynomial[Rat] =
    polynomial_constant(Rat.1) + polynomial_monomial(Nat.2, Rat.1)

/// Evaluation of the degree-two monomial over the rationals.
theorem rat_polynomial_eval_monomial_two(r: Rat, x: Rat) {
    polynomial_eval(Polynomial[Rat].monomial(Nat.2, r), x) = r * x * x
} by {
    polynomial_eval_bound_eq_coeff_eval(Polynomial[Rat].monomial(Nat.2, r), x,
        Nat.0.suc.suc.suc)
    coeff_eval(Polynomial[Rat].monomial(Nat.2, r).coeff, x, Nat.0.suc.suc.suc) =
        Polynomial[Rat].monomial(Nat.2, r).coeff(Nat.0) +
        x * coeff_eval(coeff_tail(Polynomial[Rat].monomial(Nat.2, r).coeff), x, Nat.0.suc.suc)
    polynomial_monomial_coeff_of_ne[Rat](Nat.2, r, Nat.0)
    Nat.0 != Nat.2
    Polynomial[Rat].monomial(Nat.2, r).coeff(Nat.0) = Rat.0
    coeff_eval(coeff_tail(Polynomial[Rat].monomial(Nat.2, r).coeff), x, Nat.0.suc.suc) =
        Polynomial[Rat].monomial(Nat.2, r).coeff(Nat.1) +
        x * coeff_eval(coeff_tail(coeff_tail(Polynomial[Rat].monomial(Nat.2, r).coeff)), x,
            Nat.0.suc)
    polynomial_monomial_coeff_of_ne[Rat](Nat.2, r, Nat.1)
    Nat.1 != Nat.2
    Polynomial[Rat].monomial(Nat.2, r).coeff(Nat.1) = Rat.0
    coeff_tail(coeff_tail(Polynomial[Rat].monomial(Nat.2, r).coeff))(Nat.0) =
        Polynomial[Rat].monomial(Nat.2, r).coeff(Nat.2)
    coeff_eval(coeff_tail(coeff_tail(Polynomial[Rat].monomial(Nat.2, r).coeff)), x, Nat.0.suc) =
        Polynomial[Rat].monomial(Nat.2, r).coeff(Nat.2) +
        x * coeff_eval(coeff_tail(coeff_tail(coeff_tail(
            Polynomial[Rat].monomial(Nat.2, r).coeff))), x, Nat.0)
    polynomial_monomial_coeff_self(Nat.2, r)
    Polynomial[Rat].monomial(Nat.2, r).coeff(Nat.2) = r
    coeff_eval_zero_index[Rat](coeff_tail(coeff_tail(coeff_tail(
        Polynomial[Rat].monomial(Nat.2, r).coeff))), x)
    coeff_eval(coeff_tail(coeff_tail(coeff_tail(
        Polynomial[Rat].monomial(Nat.2, r).coeff))), x, Nat.0) = Rat.0
    coeff_eval(Polynomial[Rat].monomial(Nat.2, r).coeff, x, Nat.0.suc.suc.suc) =
        Rat.0 + x * (Rat.0 + x * (r + x * Rat.0))
    Rat.0 + x * (Rat.0 + x * (r + x * Rat.0)) = r * x * x
    coeff_eval(Polynomial[Rat].monomial(Nat.2, r).coeff, x, Nat.0.suc.suc.suc) = r * x * x
    polynomial_eval_bound(Polynomial[Rat].monomial(Nat.2, r), x, Nat.0.suc.suc.suc) =
        coeff_eval(Polynomial[Rat].monomial(Nat.2, r).coeff, x, Nat.0.suc.suc.suc)
    polynomial_eval_bound(Polynomial[Rat].monomial(Nat.2, r), x, Nat.0.suc.suc.suc) = r * x * x
    polynomial_monomial_support_bounded_by_suc[Rat](Nat.2, r)
    polynomial_support_bounded_by(Polynomial[Rat].monomial(Nat.2, r), Nat.0.suc.suc.suc)
    polynomial_eval_eq_eval_bound_of_support_bounded(Polynomial[Rat].monomial(Nat.2, r), x,
        Nat.0.suc.suc.suc)
    polynomial_eval(Polynomial[Rat].monomial(Nat.2, r), x) =
        polynomial_eval_bound(Polynomial[Rat].monomial(Nat.2, r), x, Nat.0.suc.suc.suc)
    polynomial_eval(Polynomial[Rat].monomial(Nat.2, r), x) = r * x * x
}

/// The rationals are an ordered field, so every square is nonnegative.
theorem rat_square_nonneg(r: Rat) {
    Rat.0 <= r * r
} by {
    single_trichotomy(r)
    if r.is_positive {
        pos_imp_zero_lt(r)
        Rat.0 < r
        lt_imp_lte[Rat](Rat.0, r)
        Rat.0 <= r
        mul_nonnegative(r, r)
        Rat.0 <= r * r
    } else {
        not r.is_positive
        if r.is_negative {
            neg_imp_lt_zero(r)
            r < Rat.0
            lt_imp_lte[Rat](r, Rat.0)
            r <= Rat.0
            negative_of_negative[Rat](r)
            Rat.0 <= -r
            mul_nonnegative(-r, -r)
            Rat.0 <= (-r) * (-r)
            mul_neg_neg[Rat](r, r)
            -r * -r = r * r
            (-r) * (-r) = r * r
            Rat.0 <= r * r
        } else {
            not r.is_negative
            r = Rat.0
            r * r = Rat.0 * Rat.0
            Rat.0 * Rat.0 = Rat.0
            lte_refl[Rat](Rat.0)
            Rat.0 <= Rat.0
            Rat.0 <= r * r
        }
    }
}

/// The value of `X^2 + 1` at a rational point is one plus the square.
theorem rat_x_squared_plus_one_eval(x: Rat) {
    polynomial_eval(rat_x_squared_plus_one, x) = Rat.1 + x * x
} by {
    polynomial_eval_add(polynomial_constant(Rat.1), polynomial_monomial(Nat.2, Rat.1), x)
    polynomial_eval(polynomial_constant(Rat.1) + polynomial_monomial(Nat.2, Rat.1), x) =
        polynomial_eval(polynomial_constant(Rat.1), x) +
        polynomial_eval(polynomial_monomial(Nat.2, Rat.1), x)
    polynomial_eval_constant(Rat.1, x)
    polynomial_eval(polynomial_constant(Rat.1), x) = Rat.1
    rat_polynomial_eval_monomial_two(Rat.1, x)
    polynomial_eval(polynomial_monomial(Nat.2, Rat.1), x) = Rat.1 * x * x
    Rat.1 * x * x = x * x
    polynomial_eval(polynomial_constant(Rat.1) + polynomial_monomial(Nat.2, Rat.1), x) =
        Rat.1 + x * x
    polynomial_eval(rat_x_squared_plus_one, x) = Rat.1 + x * x
}

/// `X^2 + 1` has no rational root.
///
/// A rational root would satisfy `1 + x^2 = 0`; but a square is nonnegative
/// and one is positive, so `1 + x^2 > 0`.  (Alternatively, the rational root
/// theorem says a rational root of this monic integer polynomial would be an
/// integer divisor of one, i.e. `+-1`, and neither is a root.)
theorem rat_x_squared_plus_one_has_no_root {
    not exists(r: Rat) {
        polynomial_eval(rat_x_squared_plus_one, r) = Rat.0
    }
} by {
    if exists(r: Rat) {
        polynomial_eval(rat_x_squared_plus_one, r) = Rat.0
    } {
        let (r: Rat) satisfy {
            polynomial_eval(rat_x_squared_plus_one, r) = Rat.0
        }
        polynomial_eval(rat_x_squared_plus_one, r) = Rat.0
        rat_x_squared_plus_one_eval(r)
        polynomial_eval(rat_x_squared_plus_one, r) = Rat.1 + r * r
        Rat.1 + r * r = Rat.0
        rat_square_nonneg(r)
        Rat.0 <= r * r
        one_is_pos
        Rat.1.is_positive
        pos_imp_zero_lt(Rat.1)
        Rat.0 < Rat.1
        lt_add_pos(r * r, Rat.1)
        r * r < r * r + Rat.1
        lte_lt_trans[Rat](Rat.0, r * r, r * r + Rat.1)
        Rat.0 < r * r + Rat.1
        r * r + Rat.1 = Rat.1 + r * r
        Rat.0 < Rat.1 + r * r
        Rat.1 + r * r = Rat.0
        Rat.0 < Rat.0
        not_lt_self(Rat.0)
        false
    }
}

/// Evaluation of the quadratic `a X^2 + b X + c` with complex coefficients.
theorem quadratic_polynomial_eval(a: Complex, b: Complex, c: Complex, z: Complex) {
    polynomial_eval(quadratic_polynomial(a, b, c), z) = c + b * z + a * (z * z)
} by {
    polynomial_eval(quadratic_polynomial(a, b, c), z) =
        polynomial_eval(polynomial_constant(c) + polynomial_monomial(Nat.1, b) +
            polynomial_monomial(Nat.2, a), z)
    polynomial_eval_add(polynomial_constant(c) + polynomial_monomial(Nat.1, b),
        polynomial_monomial(Nat.2, a), z)
    polynomial_eval(polynomial_constant(c) + polynomial_monomial(Nat.1, b) +
        polynomial_monomial(Nat.2, a), z) =
        polynomial_eval(polynomial_constant(c) + polynomial_monomial(Nat.1, b), z) +
        polynomial_eval(polynomial_monomial(Nat.2, a), z)
    polynomial_eval_add(polynomial_constant(c), polynomial_monomial(Nat.1, b), z)
    polynomial_eval(polynomial_constant(c) + polynomial_monomial(Nat.1, b), z) =
        polynomial_eval(polynomial_constant(c), z) +
        polynomial_eval(polynomial_monomial(Nat.1, b), z)
    polynomial_eval_constant(c, z)
    polynomial_eval(polynomial_constant(c), z) = c
    polynomial_eval_monomial_one(b, z)
    polynomial_eval(polynomial_monomial(Nat.1, b), z) = b * z
    polynomial_eval_monomial_two(a, z)
    polynomial_eval(polynomial_monomial(Nat.2, a), z) = a * z * z
    polynomial_eval(quadratic_polynomial(a, b, c), z) = c + b * z + a * z * z
    mul_assoc(a, z, z)
    a * z * z = a * (z * z)
    c + b * z + a * z * z = c + b * z + a * (z * z)
    polynomial_eval(quadratic_polynomial(a, b, c), z) = c + b * z + a * (z * z)
}

/// The imaginary unit `i` is a root of `X^2 + 1`.
theorem complex_i_is_root_of_x_squared_plus_one {
    polynomial_eval(quadratic_polynomial(Complex.1, Complex.0, Complex.1), Complex.i) = Complex.0
} by {
    quadratic_polynomial_eval(Complex.1, Complex.0, Complex.1, Complex.i)
    polynomial_eval(quadratic_polynomial(Complex.1, Complex.0, Complex.1), Complex.i) =
        Complex.1 + Complex.0 * Complex.i + Complex.1 * (Complex.i * Complex.i)
    i_squared_eq_neg_one
    Complex.i * Complex.i = -Complex.1
    Complex.0 * Complex.i = Complex.0
    Complex.1 * (Complex.i * Complex.i) = Complex.1 * (-Complex.1)
    Complex.1 * (-Complex.1) = -Complex.1
    Complex.1 + Complex.0 + -Complex.1 = Complex.0
    Complex.1 + Complex.0 * Complex.i + Complex.1 * (Complex.i * Complex.i) = Complex.0
    polynomial_eval(quadratic_polynomial(Complex.1, Complex.0, Complex.1), Complex.i) = Complex.0
}

/// The conjugate `-i` is a root of `X^2 + 1`.
theorem complex_neg_i_is_root_of_x_squared_plus_one {
    polynomial_eval(quadratic_polynomial(Complex.1, Complex.0, Complex.1), -Complex.i) = Complex.0
} by {
    quadratic_polynomial_eval(Complex.1, Complex.0, Complex.1, -Complex.i)
    polynomial_eval(quadratic_polynomial(Complex.1, Complex.0, Complex.1), -Complex.i) =
        Complex.1 + Complex.0 * (-Complex.i) + Complex.1 * ((-Complex.i) * (-Complex.i))
    mul_neg_neg[Complex](Complex.i, Complex.i)
    -Complex.i * -Complex.i = Complex.i * Complex.i
    i_squared_eq_neg_one
    Complex.i * Complex.i = -Complex.1
    (-Complex.i) * (-Complex.i) = -Complex.1
    Complex.0 * (-Complex.i) = Complex.0
    Complex.1 * ((-Complex.i) * (-Complex.i)) = Complex.1 * (-Complex.1)
    Complex.1 * (-Complex.1) = -Complex.1
    Complex.1 + Complex.0 + -Complex.1 = Complex.0
    Complex.1 + Complex.0 * (-Complex.i) + Complex.1 * ((-Complex.i) * (-Complex.i)) = Complex.0
    polynomial_eval(quadratic_polynomial(Complex.1, Complex.0, Complex.1), -Complex.i) = Complex.0
}

/// The two complex roots `i` and `-i` of `X^2 + 1` are distinct.
theorem complex_i_ne_neg_i {
    Complex.i != -Complex.i
} by {
    if Complex.i = -Complex.i {
        im_i
        Complex.i.im = Real.1
        Complex.i.im = (-Complex.i).im
        neg_im(Complex.i)
        (-Complex.i).im = -Complex.i.im
        Real.1 = -Real.1
        Real.1 + Real.1 = -Real.1 + Real.1
        inverse_left[Real](Real.1)
        -Real.1 + Real.1 = Real.0
        Real.1 + Real.1 = Real.0
        two = Real.1 + Real.1
        two = Real.0
        two_nonzero
        two != Real.0
        false
    }
}

// ---------------------------------------------------------------------------
// 4. Real polynomials: continuity of evaluation and the sign-change root
// ---------------------------------------------------------------------------

/// The bounded evaluation function of a coefficient function at a fixed bound.
define coeff_eval_fn[R: Semiring](c: Nat -> R, n: Nat) -> (R -> R) {
    function(x: R) { coeff_eval(c, x, n) }
}

/// The bounded evaluation function at bound zero is the constant zero function.
theorem continuous_coeff_eval_function_zero(c: Nat -> Real) {
    continuous(coeff_eval_fn(c, Nat.0))
} by {
    forall(x: Real) {
        coeff_eval(c, x, Nat.0) = Real.0
        coeff_eval_fn(c, Nat.0, x) = Real.0
        constant[Real, Real](Real.0, x) = Real.0
        coeff_eval_fn(c, Nat.0, x) = constant[Real, Real](Real.0, x)
    }
    function_extensionality(coeff_eval_fn(c, Nat.0), constant[Real, Real](Real.0))
    coeff_eval_fn(c, Nat.0) = constant[Real, Real](Real.0)
    constant_function_is_continuous(Real.0)
    continuous(constant[Real, Real](Real.0))
    continuous(coeff_eval_fn(c, Nat.0))
}

/// Bounded polynomial evaluation in `x` is a continuous function of `x`.
///
/// Induction along the Horner bound: at bound zero the function is constant
/// zero, and each successor step is a constant plus `x` times the previous
/// step, which preserves continuity (the identity function, pointwise
/// products, and constant shifts are all continuous).
theorem continuous_coeff_eval_function(c: Nat -> Real, n: Nat) {
    continuous(coeff_eval_fn(c, n))
} by {
    define statement(k: Nat) -> Bool {
        forall(d: Nat -> Real) {
            continuous(coeff_eval_fn(d, k))
        }
    }
    forall(d: Nat -> Real) {
        continuous_coeff_eval_function_zero(d)
    }
    statement(Nat.0)
    forall(k: Nat) {
        if statement(k) {
            forall(d: Nat -> Real) {
                forall(x: Real) {
                    coeff_eval(d, x, k.suc) = d(Nat.0) + x * coeff_eval(coeff_tail(d), x, k)
                    coeff_eval_fn(d, k.suc, x) = d(Nat.0) + x * coeff_eval(coeff_tail(d), x, k)
                    coeff_eval_fn(coeff_tail(d), k, x) = coeff_eval(coeff_tail(d), x, k)
                    identity_fn[Real](x) = x
                    pointwise_mul[Real, Real](identity_fn[Real], coeff_eval_fn(coeff_tail(d), k), x) =
                        identity_fn[Real](x) * coeff_eval_fn(coeff_tail(d), k, x)
                    pointwise_mul[Real, Real](identity_fn[Real], coeff_eval_fn(coeff_tail(d), k), x) =
                        x * coeff_eval(coeff_tail(d), x, k)
                    const_add_left(d(Nat.0),
                        pointwise_mul[Real, Real](identity_fn[Real],
                            coeff_eval_fn(coeff_tail(d), k)), x) =
                        d(Nat.0) + pointwise_mul[Real, Real](identity_fn[Real],
                            coeff_eval_fn(coeff_tail(d), k), x)
                    const_add_left(d(Nat.0),
                        pointwise_mul[Real, Real](identity_fn[Real],
                            coeff_eval_fn(coeff_tail(d), k)), x) =
                        d(Nat.0) + x * coeff_eval(coeff_tail(d), x, k)
                    coeff_eval_fn(d, k.suc, x) =
                        const_add_left(d(Nat.0),
                            pointwise_mul[Real, Real](identity_fn[Real],
                                coeff_eval_fn(coeff_tail(d), k)), x)
                }
                function_extensionality(coeff_eval_fn(d, k.suc),
                    const_add_left(d(Nat.0),
                        pointwise_mul[Real, Real](identity_fn[Real],
                            coeff_eval_fn(coeff_tail(d), k))))
                coeff_eval_fn(d, k.suc) =
                    const_add_left(d(Nat.0),
                        pointwise_mul[Real, Real](identity_fn[Real],
                            coeff_eval_fn(coeff_tail(d), k)))
                identity_function_is_continuous
                continuous(identity_fn[Real])
                statement(k) = forall(e: Nat -> Real) {
                    continuous(coeff_eval_fn(e, k))
                }
                continuous(coeff_eval_fn(coeff_tail(d), k))
                continuous_pointwise_mul(identity_fn[Real], coeff_eval_fn(coeff_tail(d), k))
                continuous(pointwise_mul[Real, Real](identity_fn[Real],
                    coeff_eval_fn(coeff_tail(d), k)))
                continuous_const_add_left(d(Nat.0),
                    pointwise_mul[Real, Real](identity_fn[Real],
                        coeff_eval_fn(coeff_tail(d), k)))
                continuous(const_add_left(d(Nat.0),
                    pointwise_mul[Real, Real](identity_fn[Real],
                        coeff_eval_fn(coeff_tail(d), k))))
                continuous(coeff_eval_fn(d, k.suc))
            }
            statement(k.suc)
        }
    }
    statement(Nat.0) and forall(k: Nat) {
        statement(k) implies statement(k.suc)
    }
    alt_induction(statement)
    forall(k: Nat) { statement(k) }
    statement(n)
    forall(d: Nat -> Real) {
        continuous(coeff_eval_fn(d, n))
    }
    continuous(coeff_eval_fn(c, n))
}

/// The global evaluation function of a real polynomial.
define polynomial_eval_function[R: Semiring](p: Polynomial[R]) -> (R -> R) {
    function(x: R) { polynomial_eval(p, x) }
}

/// Evaluation at a real polynomial is a continuous function of the point.
theorem continuous_polynomial_eval_function(p: Polynomial[Real]) {
    continuous(polynomial_eval_function(p))
} by {
    polynomial_support_bound_exists(p)
    let n: Nat satisfy {
        polynomial_support_bounded_by(p, n)
    }
    forall(x: Real) {
        polynomial_eval_eq_eval_bound_of_support_bounded(p, x, n)
        polynomial_eval(p, x) = polynomial_eval_bound(p, x, n)
        polynomial_eval_bound_eq_coeff_eval(p, x, n)
        polynomial_eval_bound(p, x, n) = coeff_eval(p.coeff, x, n)
        polynomial_eval(p, x) = coeff_eval(p.coeff, x, n)
        polynomial_eval_function(p, x) = coeff_eval_fn(p.coeff, n, x)
    }
    function_extensionality(polynomial_eval_function(p), coeff_eval_fn(p.coeff, n))
    polynomial_eval_function(p) = coeff_eval_fn(p.coeff, n)
    continuous_coeff_eval_function(p.coeff, n)
    continuous(coeff_eval_fn(p.coeff, n))
    continuous(polynomial_eval_function(p))
}

/// Negation preserves continuity.
theorem continuous_neg_function(f: Real -> Real) {
    continuous(f) implies continuous(const_mul_left(-Real.1, f))
} by {
    if continuous(f) {
        continuous_const_mul_left(-Real.1, f)
        continuous(const_mul_left(-Real.1, f))
    }
}

/// A negative real number has a positive negation.
theorem real_neg_of_lt_zero_is_pos(y: Real) {
    y < Real.0 implies Real.0 < -y
} by {
    if y < Real.0 {
        neg_lt_neg[Real](y, Real.0)
        -Real.0 < -y
        neg_zero
        -Real.0 = Real.0
        Real.0 < -y
    }
}

/// A positive real number has a negative negation.
theorem real_neg_of_gt_zero_is_neg(y: Real) {
    Real.0 < y implies -y < Real.0
} by {
    if Real.0 < y {
        neg_lt_neg[Real](Real.0, y)
        -y < -Real.0
        neg_zero
        -Real.0 = Real.0
        -y < Real.0
    }
}

/// A vanishing negation means the number itself vanishes.
theorem real_neg_eq_zero_imp_eq_zero(x: Real) {
    -x = Real.0 implies x = Real.0
} by {
    if -x = Real.0 {
        -(-x) = -Real.0
        neg_neg(x)
        -(-x) = x
        neg_zero
        -Real.0 = Real.0
        x = Real.0
    }
}

/// A continuous real function taking both signs has a real zero.
///
/// This is Bolzano's theorem (the intermediate value theorem with target
/// zero) applied on the interval between the two witness points, ordered
/// either way; the reversed orientation uses the continuity of the negation.
theorem continuous_sign_change_has_zero(f: Real -> Real) {
    continuous(f) and exists(a: Real, b: Real) {
        f(a) < Real.0 and Real.0 < f(b)
    }
    implies exists(c: Real) { f(c) = Real.0 }
} by {
    if continuous(f) and exists(a: Real, b: Real) {
        f(a) < Real.0 and Real.0 < f(b)
    } {
        let (a: Real) satisfy {
            exists(b: Real) { f(a) < Real.0 and Real.0 < f(b) }
        }
        let (b: Real) satisfy {
            f(a) < Real.0 and Real.0 < f(b)
        }
        lte_or_gte(a, b)
        if a <= b {
            lt_imp_lte(f(a), Real.0)
            f(a) <= Real.0
            lt_imp_lte(Real.0, f(b))
            Real.0 <= f(b)
            intermediate_value_closed_interval(f, a, b, Real.0)
            exists(point: Real) {
                closed_interval_set(a, b).contains(point) and f(point) = Real.0
            }
            let (root: Real) satisfy {
                closed_interval_set(a, b).contains(root) and f(root) = Real.0
            }
            exists(c: Real) { f(c) = Real.0 }
        }
        if b <= a {
            continuous_const_mul_left(-Real.1, f)
            continuous(const_mul_left(-Real.1, f))
            real_neg_of_gt_zero_is_neg(f(b))
            -f(b) < Real.0
            lt_imp_lte(-f(b), Real.0)
            -f(b) <= Real.0
            mul_neg_one_left(f(b))
            -Real.1 * f(b) = -f(b)
            const_mul_left(-Real.1, f, b) = -Real.1 * f(b)
            const_mul_left(-Real.1, f, b) <= Real.0
            real_neg_of_lt_zero_is_pos(f(a))
            Real.0 < -f(a)
            lt_imp_lte(Real.0, -f(a))
            Real.0 <= -f(a)
            mul_neg_one_left(f(a))
            -Real.1 * f(a) = -f(a)
            const_mul_left(-Real.1, f, a) = -Real.1 * f(a)
            Real.0 <= const_mul_left(-Real.1, f, a)
            intermediate_value_closed_interval(const_mul_left(-Real.1, f), b, a, Real.0)
            exists(point: Real) {
                closed_interval_set(b, a).contains(point) and
                const_mul_left(-Real.1, f, point) = Real.0
            }
            let (root: Real) satisfy {
                closed_interval_set(b, a).contains(root) and
                const_mul_left(-Real.1, f, root) = Real.0
            }
            const_mul_left(-Real.1, f, root) = Real.0
            mul_neg_one_left(f(root))
            -Real.1 * f(root) = -f(root)
            const_mul_left(-Real.1, f, root) = -Real.1 * f(root)
            -f(root) = Real.0
            real_neg_eq_zero_imp_eq_zero(f(root))
            f(root) = Real.0
            exists(c: Real) { f(c) = Real.0 }
        }
        exists(c: Real) { f(c) = Real.0 }
    }
}

/// A real polynomial taking both signs has a real root: the continuity of
/// polynomial evaluation together with the sign-change root theorem.
theorem real_polynomial_sign_change_has_root(p: Polynomial[Real]) {
    exists(a: Real, b: Real) {
        polynomial_eval(p, a) < Real.0 and Real.0 < polynomial_eval(p, b)
    }
    implies exists(c: Real) { polynomial_eval(p, c) = Real.0 }
} by {
    if exists(a: Real, b: Real) {
        polynomial_eval(p, a) < Real.0 and Real.0 < polynomial_eval(p, b)
    } {
        continuous_polynomial_eval_function(p)
        continuous(polynomial_eval_function(p))
        let (a: Real) satisfy {
            exists(b: Real) { polynomial_eval(p, a) < Real.0 and Real.0 < polynomial_eval(p, b) }
        }
        let (b: Real) satisfy {
            polynomial_eval(p, a) < Real.0 and Real.0 < polynomial_eval(p, b)
        }
        polynomial_eval_function(p, a) = polynomial_eval(p, a)
        polynomial_eval_function(p, b) = polynomial_eval(p, b)
        polynomial_eval_function(p, a) < Real.0
        Real.0 < polynomial_eval_function(p, b)
        polynomial_eval_function(p, a) < Real.0 and Real.0 < polynomial_eval_function(p, b)
        exists(a0: Real, b0: Real) {
            polynomial_eval_function(p, a0) < Real.0 and Real.0 < polynomial_eval_function(p, b0)
        }
        continuous_sign_change_has_zero(polynomial_eval_function(p))
        exists(c: Real) { polynomial_eval_function(p, c) = Real.0 }
        let (c: Real) satisfy {
            polynomial_eval_function(p, c) = Real.0
        }
        polynomial_eval_function(p, c) = Real.0
        polynomial_eval_function(p, c) = polynomial_eval(p, c)
        polynomial_eval(p, c) = Real.0
        exists(c0: Real) {
            c0 = c and polynomial_eval(p, c0) = Real.0
        }
        exists(c0: Real) { polynomial_eval(p, c0) = Real.0 }
    }
}

// ---------------------------------------------------------------------------
// 4b. Odd degree: statement and status
// ---------------------------------------------------------------------------

// The classical consequence of the two theorems above:
//
//     every real polynomial of odd degree with nonzero leading coefficient
//     has a real root.
//
// In the support-bound formulation, a polynomial of degree `2k + 1` is
// supported below `2k + 2` with nonzero coefficient at `2k + 1`.  The
// statement is:
//
//     theorem real_polynomial_odd_degree_has_root(p: Polynomial[Real], k: Nat) {
//         polynomial_support_bounded_by(p, k.suc.suc.suc) and
//             p.coeff(k.suc.suc) != Real.0
//         implies exists(x: Real) { polynomial_eval(p, x) = Real.0 }
//     }
//
// By `continuous_polynomial_eval_function` and
// `continuous_sign_change_has_zero` it suffices to show that the value of an
// odd-degree polynomial is negative somewhere and positive somewhere.  That
// is the leading-term dominance lemma: for a leading coefficient `a_n != 0`,
// the value `a_n x^n` dominates the lower terms once `|x|` is large, so the
// polynomial has the sign of `a_n x^n` at `+/- infinity`; for odd `n` the two
// signs differ.  Formally this needs bounds like
// `|a_i| * |x|^i <= (|a_n| / 2) * |x|^n` for all lower `i` on a large
// interval, together with triangle-inequality and power-monotonicity
// reasoning over the reals; that machinery is not yet in the library, so the
// theorem is recorded here as a statement only.

// ---------------------------------------------------------------------------
// 5. Eisenstein's criterion (statement)
// ---------------------------------------------------------------------------

// Eisenstein's criterion: a polynomial
//
//     a_n X^n + a_{n-1} X^{n-1} + ... + a_1 X + a_0
//
// with integer coefficients is irreducible over the rationals when some prime
// `p` divides every coefficient except the leading one, and `p^2` does not
// divide the constant term `a_0`:
//
//     p | a_i for all i < n,   p does not divide a_n,   p^2 does not divide a_0
//     implies irreducibility of the rationalised polynomial.
//
// The classical proof: a factorisation over the rationals is (after clearing
// denominators) a factorisation over the integers (Gauss's lemma); reducing
// modulo `p` the polynomial becomes `a_n X^n`, so each factor would be a
// monomial modulo `p`, forcing all non-leading coefficients of the factors to
// be divisible by `p`; then the constant terms of the factors are both
// divisible by `p`, making the product's constant term divisible by `p^2`,
// contradicting the hypothesis.  This needs the library's polynomial
// multiplication, Gauss's lemma (primitivity), and reduction modulo a prime,
// none of which are available yet, so the criterion is recorded here as a
// statement only.

