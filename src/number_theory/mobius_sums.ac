from nat import Nat
from int import Int, neg_zero, neg_distrib, add_comm, add_assoc, add_zero_left,
    add_zero_right
from list import List, map, sum, remove_one_cons_eq, remove_one_cons_neq
numerals Nat
numerals Int

/// The sum over a list splits by a filter and a complementary filter.
theorem sum_filter_split2(l: List[Nat], p: Nat -> Bool, q: Nat -> Bool,
    f: Nat -> Int) {
    (forall(x: Nat) { l.contains(x) implies (q(x) = not p(x)) })
        implies sum(map(l, f)) =
            sum(map(l.filter(p), f)) + sum(map(l.filter(q), f))
} by {
    define pr(ls: List[Nat]) -> Bool {
        (forall(x: Nat) { ls.contains(x) implies (q(x) = not p(x)) })
            implies sum(map(ls, f)) =
                sum(map(ls.filter(p), f)) + sum(map(ls.filter(q), f))
    }
    if forall(x: Nat) { l.contains(x) implies (q(x) = not p(x)) } {
        map(List.nil[Nat], f) = List.nil[Int]
        sum(List.nil[Int]) = Int.0
        map(List.nil[Nat].filter(p), f) = List.nil[Int]
        sum(List.nil[Int]) = Int.0
        map(List.nil[Nat].filter(q), f) = List.nil[Int]
        Int.0 + Int.0 = Int.0
        pr(List.nil[Nat])
        forall(head: Nat, tail: List[Nat]) {
            if pr(tail) {
                if forall(x: Nat) { List.cons(head, tail).contains(x) implies (q(x) = not p(x)) } {
                    forall(x: Nat) {
                        if tail.contains(x) {
                            if head = x {
                                List.cons(head, tail).contains(x) = true
                                List.cons(head, tail).contains(x)
                            } else {
                                head != x
                                List.cons(head, tail).contains(x) = tail.contains(x)
                                List.cons(head, tail).contains(x)
                            }
                            List.cons(head, tail).contains(x) implies (q(x) = not p(x))
                            q(x) = not p(x)
                        }
                        tail.contains(x) implies (q(x) = not p(x))
                    }
                    pr(tail)
                    pr(tail) = ((forall(x: Nat) { tail.contains(x) implies (q(x) = not p(x)) }) implies sum(map(tail, f)) = sum(map(tail.filter(p), f)) + sum(map(tail.filter(q), f)))
                    forall(x: Nat) { tail.contains(x) implies (q(x) = not p(x)) }
                    sum(map(tail, f)) = sum(map(tail.filter(p), f)) + sum(map(tail.filter(q), f))
                    List.cons(head, tail).contains(head)
                    q(head) = not p(head)
                    if p(head) {
                        List.cons(head, tail).filter(p) = List.cons(head, tail.filter(p))
                        List.cons(head, tail).filter(q) = tail.filter(q)
                        q(head) = false
                        map(List.cons(head, tail), f) = List.cons(f(head), map(tail, f))
                        sum(List.cons(f(head), map(tail, f))) = f(head) + sum(map(tail, f))
                        sum(map(List.cons(head, tail), f)) = f(head) + sum(map(tail, f))
                        map(List.cons(head, tail.filter(p)), f) =
                            List.cons(f(head), map(tail.filter(p), f))
                        sum(List.cons(f(head), map(tail.filter(p), f))) =
                            f(head) + sum(map(tail.filter(p), f))
                        sum(map(List.cons(head, tail).filter(p), f)) =
                            f(head) + sum(map(tail.filter(p), f))
                        f(head) + (sum(map(tail.filter(p), f)) + sum(map(tail.filter(q), f))) =
                            (f(head) + sum(map(tail.filter(p), f))) + sum(map(tail.filter(q), f))
                        f(head) + sum(map(tail, f)) =
                            (f(head) + sum(map(tail.filter(p), f))) + sum(map(tail.filter(q), f))
                        sum(map(List.cons(head, tail), f)) =
                            sum(map(List.cons(head, tail).filter(p), f)) +
                            sum(map(List.cons(head, tail).filter(q), f))
                    } else {
                        List.cons(head, tail).filter(p) = tail.filter(p)
                        List.cons(head, tail).filter(q) = List.cons(head, tail.filter(q))
                        q(head) = true
                        map(List.cons(head, tail), f) = List.cons(f(head), map(tail, f))
                        sum(List.cons(f(head), map(tail, f))) = f(head) + sum(map(tail, f))
                        sum(map(List.cons(head, tail), f)) = f(head) + sum(map(tail, f))
                        map(List.cons(head, tail.filter(q)), f) =
                            List.cons(f(head), map(tail.filter(q), f))
                        sum(List.cons(f(head), map(tail.filter(q), f))) =
                            f(head) + sum(map(tail.filter(q), f))
                        sum(map(List.cons(head, tail).filter(q), f)) =
                            f(head) + sum(map(tail.filter(q), f))
                        add_assoc(sum(map(tail.filter(p), f)), f(head),
                            sum(map(tail.filter(q), f)))
                        sum(map(tail.filter(p), f)) + (f(head) + sum(map(tail.filter(q), f))) =
                            (sum(map(tail.filter(p), f)) + f(head)) + sum(map(tail.filter(q), f))
                        add_comm(sum(map(tail.filter(p), f)), f(head))
                        sum(map(tail.filter(p), f)) + f(head) =
                            f(head) + sum(map(tail.filter(p), f))
                        sum(map(tail.filter(p), f)) + (f(head) + sum(map(tail.filter(q), f))) =
                            (f(head) + sum(map(tail.filter(p), f))) + sum(map(tail.filter(q), f))
                        add_assoc(f(head), sum(map(tail.filter(p), f)),
                            sum(map(tail.filter(q), f)))
                        f(head) + (sum(map(tail.filter(p), f)) + sum(map(tail.filter(q), f))) =
                            (f(head) + sum(map(tail.filter(p), f))) + sum(map(tail.filter(q), f))
                        sum(map(tail.filter(p), f)) + (f(head) + sum(map(tail.filter(q), f))) =
                            f(head) + (sum(map(tail.filter(p), f)) + sum(map(tail.filter(q), f)))
                        sum(map(tail.filter(p), f)) + (f(head) + sum(map(tail.filter(q), f))) =
                            f(head) + sum(map(tail, f))
                        sum(map(List.cons(head, tail), f)) =
                            sum(map(List.cons(head, tail).filter(p), f)) +
                            sum(map(List.cons(head, tail).filter(q), f))
                    }
                }
                pr(List.cons(head, tail))
            }
        }
        forall(head: Nat, tail: List[Nat]) {
            pr(tail) implies pr(List.cons(head, tail))
        }
        pr(List.nil[Nat]) and forall(head: Nat, tail: List[Nat]) {
            pr(tail) implies pr(List.cons(head, tail))
        }
        List.induction(pr)
        forall(ls: List[Nat]) { pr(ls) }
        pr(l)
    }
}

/// Filtering out elements where `f` vanishes does not change the sum.
theorem sum_filter_zero_removed(l: List[Nat], pred: Nat -> Bool, f: Nat -> Int) {
    (forall(x: Nat) { l.contains(x) and not pred(x) implies f(x) = Int.0 })
        implies sum(map(l, f)) = sum(map(l.filter(pred), f))
} by {
    define pr(ls: List[Nat]) -> Bool {
        (forall(x: Nat) { ls.contains(x) and not pred(x) implies f(x) = Int.0 })
            implies sum(map(ls, f)) = sum(map(ls.filter(pred), f))
    }
    if forall(x: Nat) { l.contains(x) and not pred(x) implies f(x) = Int.0 } {
        map(List.nil[Nat], f) = List.nil[Int]
        sum(List.nil[Int]) = Int.0
        map(List.nil[Nat].filter(pred), f) = List.nil[Int]
        pr(List.nil[Nat])
        forall(head: Nat, tail: List[Nat]) {
            if pr(tail) {
                if forall(x: Nat) { List.cons(head, tail).contains(x) and not pred(x) implies f(x) = Int.0 } {
                    forall(x: Nat) {
                        if tail.contains(x) and not pred(x) {
                            if head = x {
                                List.cons(head, tail).contains(x) = true
                                List.cons(head, tail).contains(x)
                            } else {
                                head != x
                                List.cons(head, tail).contains(x) = tail.contains(x)
                                List.cons(head, tail).contains(x)
                            }
                            List.cons(head, tail).contains(x) and not pred(x) implies f(x) = Int.0
                            f(x) = Int.0
                        }
                        tail.contains(x) and not pred(x) implies f(x) = Int.0
                    }
                    pr(tail)
                    pr(tail) = ((forall(x: Nat) {
                        tail.contains(x) and not pred(x) implies f(x) = Int.0
                    }) implies sum(map(tail, f)) = sum(map(tail.filter(pred), f)))
                    forall(x: Nat) { tail.contains(x) and not pred(x) implies f(x) = Int.0 }
                    sum(map(tail, f)) = sum(map(tail.filter(pred), f))
                    if pred(head) {
                        List.cons(head, tail).filter(pred) = List.cons(head, tail.filter(pred))
                        map(List.cons(head, tail), f) = List.cons(f(head), map(tail, f))
                        sum(List.cons(f(head), map(tail, f))) = f(head) + sum(map(tail, f))
                        sum(map(List.cons(head, tail), f)) = f(head) + sum(map(tail, f))
                        map(List.cons(head, tail.filter(pred)), f) =
                            List.cons(f(head), map(tail.filter(pred), f))
                        sum(List.cons(f(head), map(tail.filter(pred), f))) =
                            f(head) + sum(map(tail.filter(pred), f))
                        sum(map(List.cons(head, tail).filter(pred), f)) =
                            f(head) + sum(map(tail.filter(pred), f))
                        f(head) + sum(map(tail, f)) = f(head) + sum(map(tail.filter(pred), f))
                        sum(map(List.cons(head, tail), f)) =
                            sum(map(List.cons(head, tail).filter(pred), f))
                    } else {
                        not pred(head)
                        List.cons(head, tail).filter(pred) = tail.filter(pred)
                        List.cons(head, tail).contains(head) and not pred(head)
                        f(head) = Int.0
                        map(List.cons(head, tail), f) = List.cons(f(head), map(tail, f))
                        sum(List.cons(f(head), map(tail, f))) = f(head) + sum(map(tail, f))
                        sum(map(List.cons(head, tail), f)) = f(head) + sum(map(tail, f))
                        add_zero_left(sum(map(tail, f)))
                        Int.0 + sum(map(tail, f)) = sum(map(tail, f))
                        sum(map(List.cons(head, tail), f)) = sum(map(tail, f))
                        sum(map(List.cons(head, tail), f)) = sum(map(tail.filter(pred), f))
                        sum(map(List.cons(head, tail), f)) =
                            sum(map(List.cons(head, tail).filter(pred), f))
                    }
                }
                pr(List.cons(head, tail))
            }
        }
        forall(head: Nat, tail: List[Nat]) {
            pr(tail) implies pr(List.cons(head, tail))
        }
        pr(List.nil[Nat]) and forall(head: Nat, tail: List[Nat]) {
            pr(tail) implies pr(List.cons(head, tail))
        }
        List.induction(pr)
        forall(ls: List[Nat]) { pr(ls) }
        pr(l)
    }
}

/// The sum of negated values is the negation of the sum.
theorem int_sum_map_neg(l: List[Nat], f: Nat -> Int) {
    sum(map(l, function(x: Nat) { -f(x) })) = -sum(map(l, f))
} by {
    let g: Nat -> Int = function(x: Nat) { -f(x) }
    define pr(ls: List[Nat]) -> Bool {
        sum(map(ls, g)) = -sum(map(ls, f))
    }
    map(List.nil[Nat], g) = List.nil[Int]
    sum(List.nil[Int]) = Int.0
    sum(map(List.nil[Nat], g)) = Int.0
    sum(map(List.nil[Nat], f)) = Int.0
    neg_zero
    -Int.0 = Int.0
    -sum(map(List.nil[Nat], f)) = Int.0
    pr(List.nil[Nat])
    forall(head: Nat, tail: List[Nat]) {
        if pr(tail) {
            pr(tail) = (sum(map(tail, g)) = -sum(map(tail, f)))
            g(head) = -f(head)
            map(List.cons(head, tail), g) = List.cons(g(head), map(tail, g))
            sum(List.cons(g(head), map(tail, g))) = g(head) + sum(map(tail, g))
            sum(map(List.cons(head, tail), g)) = g(head) + sum(map(tail, g))
            map(List.cons(head, tail), f) = List.cons(f(head), map(tail, f))
            sum(List.cons(f(head), map(tail, f))) = f(head) + sum(map(tail, f))
            sum(map(List.cons(head, tail), f)) = f(head) + sum(map(tail, f))
            neg_distrib(f(head), sum(map(tail, f)))
            -(f(head) + sum(map(tail, f))) = -f(head) + -sum(map(tail, f))
            g(head) + sum(map(tail, g)) = -f(head) + -sum(map(tail, f))
            sum(map(List.cons(head, tail), g)) =
                -(f(head) + sum(map(tail, f)))
            sum(map(List.cons(head, tail), g)) =
                -sum(map(List.cons(head, tail), f))
            pr(List.cons(head, tail))
        }
    }
    forall(head: Nat, tail: List[Nat]) {
        pr(tail) implies pr(List.cons(head, tail))
    }
    pr(List.nil[Nat]) and forall(head: Nat, tail: List[Nat]) {
        pr(tail) implies pr(List.cons(head, tail))
    }
    List.induction(pr)
    forall(ls: List[Nat]) { pr(ls) }
    pr(l)
    pr(l) = (sum(map(l, g)) = -sum(map(l, f)))
    g = function(x: Nat) { -f(x) }
    map(l, g) = map(l, function(x: Nat) { -f(x) })
    sum(map(l, g)) = sum(map(l, function(x: Nat) { -f(x) }))
    sum(map(l, function(x: Nat) { -f(x) })) = -sum(map(l, f))
}

/// Removing an occurrence from a list decreases its length by one.
theorem remove_one_length_suc[T](list: List[T], item: T) {
    list.contains(item) implies list.remove_one(item).length.suc = list.length
} by {
    define pr(l: List[T]) -> Bool {
        l.contains(item) implies l.remove_one(item).length.suc = l.length
    }
    pr(List.nil[T])
    forall(head: T, tail: List[T]) {
        if pr(tail) {
            pr(tail) = (tail.contains(item) implies tail.remove_one(item).length.suc = tail.length)
            if List.cons(head, tail).contains(item) {
                if head = item {
                    remove_one_cons_eq(head, tail)
                    List.cons(head, tail).remove_one(item) = tail
                    List.cons(head, tail).remove_one(item).length.suc = tail.length.suc
                    List.cons(head, tail).length = tail.length.suc
                    List.cons(head, tail).remove_one(item).length.suc = List.cons(head, tail).length
                } else {
                    head != item
                    tail.contains(item)
                    tail.remove_one(item).length.suc = tail.length
                    remove_one_cons_neq(head, tail, item)
                    List.cons(head, tail).remove_one(item) = List.cons(head, tail.remove_one(item))
                    List.cons(head, tail.remove_one(item)).length =
                        tail.remove_one(item).length.suc
                    List.cons(head, tail).remove_one(item).length =
                        List.cons(head, tail.remove_one(item)).length
                    List.cons(head, tail).remove_one(item).length =
                        tail.remove_one(item).length.suc
                    tail.remove_one(item).length.suc = tail.length
                    List.cons(head, tail).remove_one(item).length = tail.length
                    List.cons(head, tail).remove_one(item).length.suc = tail.length.suc
                    List.cons(head, tail).length = tail.length.suc
                    List.cons(head, tail).remove_one(item).length.suc = List.cons(head, tail).length
                }
            }
            pr(List.cons(head, tail))
        }
    }
    forall(head: T, tail: List[T]) {
        pr(tail) implies pr(List.cons(head, tail))
    }
    pr(List.nil[T]) and forall(head: T, tail: List[T]) {
        pr(tail) implies pr(List.cons(head, tail))
    }
    List.induction(pr)
    forall(ls: List[T]) { pr(ls) }
    pr(list)
}

/// True when a natural is not divisible by `p`.
define not_divides_pred(p: Nat) -> (Nat -> Bool) {
    function(d: Nat) { not p.divides(d) }
}

/// True when a natural is divisible by `p`.
define divides_pred(p: Nat) -> (Nat -> Bool) {
    function(d: Nat) { p.divides(d) }
}

/// The sum over a list splits by `p`-divisibility of its elements.
theorem sum_split_by_p(l: List[Nat], p: Nat, f: Nat -> Int) {
    sum(map(l, f)) =
        sum(map(l.filter(not_divides_pred(p)), f)) + sum(map(l.filter(divides_pred(p)), f))
} by {
    define pr(ls: List[Nat]) -> Bool {
        sum(map(ls, f)) =
            sum(map(ls.filter(not_divides_pred(p)), f)) +
            sum(map(ls.filter(divides_pred(p)), f))
    }
    map(List.nil[Nat], f) = List.nil[Int]
    sum(List.nil[Int]) = Int.0
    map(List.nil[Nat].filter(not_divides_pred(p)), f) = List.nil[Int]
    sum(List.nil[Int]) = Int.0
    map(List.nil[Nat].filter(divides_pred(p)), f) = List.nil[Int]
    Int.0 + Int.0 = Int.0
    pr(List.nil[Nat])
    forall(head: Nat, tail: List[Nat]) {
        if pr(tail) {
            pr(tail) = (sum(map(tail, f)) =
                sum(map(tail.filter(not_divides_pred(p)), f)) +
                sum(map(tail.filter(divides_pred(p)), f)))
            sum(map(tail, f)) =
                sum(map(tail.filter(not_divides_pred(p)), f)) +
                sum(map(tail.filter(divides_pred(p)), f))
            not_divides_pred(p)(head) = (not p.divides(head))
            divides_pred(p)(head) = p.divides(head)
            if p.divides(head) {
                not_divides_pred(p)(head) = false
                divides_pred(p)(head) = true
                List.cons(head, tail).filter(not_divides_pred(p)) =
                    tail.filter(not_divides_pred(p))
                List.cons(head, tail).filter(divides_pred(p)) =
                    List.cons(head, tail.filter(divides_pred(p)))
                map(List.cons(head, tail), f) = List.cons(f(head), map(tail, f))
                sum(List.cons(f(head), map(tail, f))) = f(head) + sum(map(tail, f))
                sum(map(List.cons(head, tail), f)) = f(head) + sum(map(tail, f))
                map(List.cons(head, tail.filter(divides_pred(p))), f) =
                    List.cons(f(head), map(tail.filter(divides_pred(p)), f))
                sum(List.cons(f(head), map(tail.filter(divides_pred(p)), f))) =
                    f(head) + sum(map(tail.filter(divides_pred(p)), f))
                sum(map(List.cons(head, tail).filter(divides_pred(p)), f)) =
                    f(head) + sum(map(tail.filter(divides_pred(p)), f))
                add_assoc(sum(map(tail.filter(not_divides_pred(p)), f)), f(head),
                    sum(map(tail.filter(divides_pred(p)), f)))
                sum(map(tail.filter(not_divides_pred(p)), f)) +
                    (f(head) + sum(map(tail.filter(divides_pred(p)), f))) =
                    (sum(map(tail.filter(not_divides_pred(p)), f)) + f(head)) +
                    sum(map(tail.filter(divides_pred(p)), f))
                add_comm(sum(map(tail.filter(not_divides_pred(p)), f)), f(head))
                sum(map(tail.filter(not_divides_pred(p)), f)) + f(head) =
                    f(head) + sum(map(tail.filter(not_divides_pred(p)), f))
                sum(map(tail.filter(not_divides_pred(p)), f)) +
                    (f(head) + sum(map(tail.filter(divides_pred(p)), f))) =
                    (f(head) + sum(map(tail.filter(not_divides_pred(p)), f))) +
                    sum(map(tail.filter(divides_pred(p)), f))
                add_assoc(f(head), sum(map(tail.filter(not_divides_pred(p)), f)),
                    sum(map(tail.filter(divides_pred(p)), f)))
                f(head) + (sum(map(tail.filter(not_divides_pred(p)), f)) +
                    sum(map(tail.filter(divides_pred(p)), f))) =
                    (f(head) + sum(map(tail.filter(not_divides_pred(p)), f))) +
                    sum(map(tail.filter(divides_pred(p)), f))
                sum(map(tail.filter(not_divides_pred(p)), f)) +
                    (f(head) + sum(map(tail.filter(divides_pred(p)), f))) =
                    f(head) + (sum(map(tail.filter(not_divides_pred(p)), f)) +
                        sum(map(tail.filter(divides_pred(p)), f)))
                sum(map(tail.filter(not_divides_pred(p)), f)) +
                    (f(head) + sum(map(tail.filter(divides_pred(p)), f))) =
                    f(head) + sum(map(tail, f))
                sum(map(List.cons(head, tail), f)) =
                    sum(map(List.cons(head, tail).filter(not_divides_pred(p)), f)) +
                    sum(map(List.cons(head, tail).filter(divides_pred(p)), f))
            } else {
                not p.divides(head)
                not_divides_pred(p)(head) = true
                divides_pred(p)(head) = false
                List.cons(head, tail).filter(not_divides_pred(p)) =
                    List.cons(head, tail.filter(not_divides_pred(p)))
                List.cons(head, tail).filter(divides_pred(p)) =
                    tail.filter(divides_pred(p))
                map(List.cons(head, tail), f) = List.cons(f(head), map(tail, f))
                sum(List.cons(f(head), map(tail, f))) = f(head) + sum(map(tail, f))
                sum(map(List.cons(head, tail), f)) = f(head) + sum(map(tail, f))
                map(List.cons(head, tail.filter(not_divides_pred(p))), f) =
                    List.cons(f(head), map(tail.filter(not_divides_pred(p)), f))
                sum(List.cons(f(head), map(tail.filter(not_divides_pred(p)), f))) =
                    f(head) + sum(map(tail.filter(not_divides_pred(p)), f))
                sum(map(List.cons(head, tail).filter(not_divides_pred(p)), f)) =
                    f(head) + sum(map(tail.filter(not_divides_pred(p)), f))
                f(head) + (sum(map(tail.filter(not_divides_pred(p)), f)) +
                    sum(map(tail.filter(divides_pred(p)), f))) =
                    (f(head) + sum(map(tail.filter(not_divides_pred(p)), f))) +
                    sum(map(tail.filter(divides_pred(p)), f))
                f(head) + sum(map(tail, f)) =
                    (f(head) + sum(map(tail.filter(not_divides_pred(p)), f))) +
                    sum(map(tail.filter(divides_pred(p)), f))
                sum(map(List.cons(head, tail), f)) =
                    sum(map(List.cons(head, tail).filter(not_divides_pred(p)), f)) +
                    sum(map(List.cons(head, tail).filter(divides_pred(p)), f))
            }
            pr(List.cons(head, tail))
        }
    }
    forall(head: Nat, tail: List[Nat]) {
        pr(tail) implies pr(List.cons(head, tail))
    }
    pr(List.nil[Nat]) and forall(head: Nat, tail: List[Nat]) {
        pr(tail) implies pr(List.cons(head, tail))
    }
    List.induction(pr)
    forall(ls: List[Nat]) { pr(ls) }
    pr(l)
}
