from nat import Nat, mul_to_zero, divides_self, divides_lte, mul_cancel_left,
    lt_suc_right, lt_imp_lte_suc, lte_and_lt, lt_not_ref, lt_trans, lt_suc
from int import Int, neg_neg, one_neq_zero, add_comm, add_neg, mul_comm
from list import List, map, sum, product, is_permutation, permutation_preserves_length,
    add_length, product_append, nil_count_zero
from algebra.ring.ring import Ring, alternating_sign, alternating_sign_zero,
    alternating_sign_suc, alternating_sign_eq_neg_one_pow, alternating_sign_parity,
    mul_neg_left, mul_zero_left
from number_theory.factorisation import prime_factorisation, prime_factorisation_product,
    prime_factorisation_all_prime, prime_factorisation_unique, all_prime,
    all_prime_nil, all_prime_cons_intro, all_prime_append, all_prime_product_nonzero,
    all_prime_product_eq_one_imp_nil, no_proper_divisor_imp_prime
from number_theory.mobius_inversion import prime_factorisation_one,
    prime_factorisation_prime, alternating_sign_add, int_alternating_sign_nonzero
from number_theory.zsigmondy import divides_suc_pair_imp_one
from number_theory.divisor_sum import divisor_list, divisor_list_one, divisor_list_prime,
    divisor_list_contains_of, divisors_up_to, divisors_up_to_suc_yes, divisors_up_to_suc_no,
    divisors_up_to_zero, divisors_up_to_one
from data.nat.nat_square import is_square, is_square_intro, is_square_witness,
    one_is_square
numerals Nat
numerals Int

/// The number of prime factors of `n` counted with multiplicity (the function
/// `Omega(n)`). For `n >= 1` this is the length of the prime factorisation;
/// for zero the empty placeholder factorisation makes it zero.
define nat_prime_omega(n: Nat) -> Nat {
    prime_factorisation(n).length
}

/// The Liouville function `lambda(n) = (-1)^Omega(n)`, where `Omega(n)` is the
/// number of prime factors of `n` counted with multiplicity.
define nat_liouville(n: Nat) -> Int {
    alternating_sign[Int](nat_prime_omega(n))
}

/// The alternating sign takes only the values one and minus one.
theorem int_alternating_sign_one_or_neg_one(k: Nat) {
    alternating_sign[Int](k) = Int.1 or alternating_sign[Int](k) = -Int.1
} by {
    define p(n: Nat) -> Bool {
        alternating_sign[Int](n) = Int.1 or alternating_sign[Int](n) = -Int.1
    }
    alternating_sign_zero[Int]
    alternating_sign[Int](Nat.0) = Int.1
    p(Nat.0)
    forall(n: Nat) {
        if p(n) {
            p(n) = (alternating_sign[Int](n) = Int.1 or alternating_sign[Int](n) = -Int.1)
            alternating_sign_suc[Int](n)
            alternating_sign[Int](n.suc) = -alternating_sign[Int](n)
            if alternating_sign[Int](n) = Int.1 {
                alternating_sign[Int](n.suc) = -Int.1
                p(n.suc)
            } else {
                alternating_sign[Int](n) = -Int.1
                alternating_sign[Int](n.suc) = -(-Int.1)
                neg_neg(Int.1)
                -(-Int.1) = Int.1
                alternating_sign[Int](n.suc) = Int.1
                p(n.suc)
            }
            p(n.suc)
        }
    }
    p(Nat.0) and forall(n: Nat) {
        p(n) implies p(n.suc)
    }
    Nat.induction(p)
    p(k)
}

/// The Liouville function takes only the values one and minus one.
theorem nat_liouville_one_or_neg_one(n: Nat) {
    nat_liouville(n) = Int.1 or nat_liouville(n) = -Int.1
} by {
    int_alternating_sign_one_or_neg_one(nat_prime_omega(n))
    alternating_sign[Int](nat_prime_omega(n)) = Int.1 or
        alternating_sign[Int](nat_prime_omega(n)) = -Int.1
    nat_liouville(n) = alternating_sign[Int](nat_prime_omega(n))
    nat_liouville(n) = Int.1 or nat_liouville(n) = -Int.1
}

/// `lambda(1) = 1`.
theorem nat_liouville_one {
    nat_liouville(Nat.1) = Int.1
} by {
    prime_factorisation_one
    prime_factorisation(Nat.1) = List.nil[Nat]
    List.nil[Nat].length = Nat.0
    alternating_sign_zero[Int]
    alternating_sign[Int](Nat.0) = Int.1
    nat_prime_omega(Nat.1) = prime_factorisation(Nat.1).length
    nat_prime_omega(Nat.1) = Nat.0
    alternating_sign[Int](nat_prime_omega(Nat.1)) = alternating_sign[Int](Nat.0)
    alternating_sign[Int](nat_prime_omega(Nat.1)) = Int.1
    nat_liouville(Nat.1) = alternating_sign[Int](nat_prime_omega(Nat.1))
    nat_liouville(Nat.1) = Int.1
}

/// Two is prime.
theorem nat_two_prime_local {
    Nat.2.is_prime
} by {
    Nat.1 < Nat.2
    forall(k: Nat) {
        if Nat.1 < k and k < Nat.2 {
            lt_imp_lte_suc(Nat.1, k)
            Nat.2 <= k
            lte_and_lt(Nat.2, k, Nat.2)
            Nat.2 < Nat.2
            lt_not_ref(Nat.2)
            false
        }
    }
    no_proper_divisor_imp_prime(Nat.2)
}

/// `lambda(2) = -1`.
theorem nat_liouville_two {
    nat_liouville(Nat.2) = -Int.1
} by {
    nat_two_prime_local
    Nat.2.is_prime
    prime_factorisation_prime(Nat.2)
    prime_factorisation(Nat.2) = List.singleton(Nat.2)
    List.singleton(Nat.2).length = Nat.1
    alternating_sign_suc[Int](Nat.0)
    alternating_sign[Int](Nat.0.suc) = -alternating_sign[Int](Nat.0)
    alternating_sign_zero[Int]
    alternating_sign[Int](Nat.0) = Int.1
    Nat.0.suc = Nat.1
    alternating_sign[Int](Nat.1) = -Int.1
    nat_prime_omega(Nat.2) = prime_factorisation(Nat.2).length
    nat_prime_omega(Nat.2) = Nat.1
    alternating_sign[Int](nat_prime_omega(Nat.2)) = alternating_sign[Int](Nat.1)
    alternating_sign[Int](nat_prime_omega(Nat.2)) = -Int.1
    nat_liouville(Nat.2) = alternating_sign[Int](nat_prime_omega(Nat.2))
    nat_liouville(Nat.2) = -Int.1
}

/// The number of prime factors of a product of positive naturals is the sum
/// of the numbers of prime factors of the factors: `Omega(mn) = Omega(m) + Omega(n)`.
theorem nat_prime_omega_mul(a: Nat, b: Nat) {
    a != Nat.0 and b != Nat.0 implies
        nat_prime_omega(a * b) = nat_prime_omega(a) + nat_prime_omega(b)
} by {
    if a != Nat.0 and b != Nat.0 {
        Nat.1 <= a
        Nat.1 <= b
        let fa: List[Nat] = prime_factorisation(a)
        let fb: List[Nat] = prime_factorisation(b)
        prime_factorisation_product(a)
        prime_factorisation_product(b)
        prime_factorisation_all_prime(a)
        prime_factorisation_all_prime(b)
        all_prime(fa)
        all_prime(fb)
        product[Nat](fa) = a
        product[Nat](fb) = b
        let combined: List[Nat] = fa + fb
        all_prime_append(fa, fb)
        all_prime(combined)
        product_append[Nat](fa, fb)
        product[Nat](combined) = product[Nat](fa) * product[Nat](fb)
        product[Nat](combined) = a * b
        if a * b = Nat.0 {
            mul_to_zero(a, b)
            a = Nat.0 or b = Nat.0
            false
        }
        a * b != Nat.0
        Nat.1 <= a * b
        prime_factorisation_product(a * b)
        prime_factorisation_all_prime(a * b)
        all_prime(prime_factorisation(a * b))
        product[Nat](prime_factorisation(a * b)) = a * b
        prime_factorisation_unique(combined, prime_factorisation(a * b))
        is_permutation(combined, prime_factorisation(a * b))
        permutation_preserves_length(combined, prime_factorisation(a * b))
        combined.length = prime_factorisation(a * b).length
        add_length(fa, fb)
        combined.length = fa.length + fb.length
        prime_factorisation(a * b).length = fa.length + fb.length
        nat_prime_omega(a * b) = prime_factorisation(a * b).length
        nat_prime_omega(a) = prime_factorisation(a).length
        nat_prime_omega(b) = prime_factorisation(b).length
        nat_prime_omega(a * b) = nat_prime_omega(a) + nat_prime_omega(b)
    }
}

/// The Liouville function is completely multiplicative on positive arguments:
/// `lambda(mn) = lambda(m) * lambda(n)` for `m, n > 0`.
theorem nat_liouville_mul(a: Nat, b: Nat) {
    a != Nat.0 and b != Nat.0 implies nat_liouville(a * b) = nat_liouville(a) * nat_liouville(b)
} by {
    if a != Nat.0 and b != Nat.0 {
        nat_prime_omega_mul(a, b)
        nat_prime_omega(a * b) = nat_prime_omega(a) + nat_prime_omega(b)
        alternating_sign_add[Int](nat_prime_omega(a), nat_prime_omega(b))
        alternating_sign[Int](nat_prime_omega(a) + nat_prime_omega(b)) =
            alternating_sign[Int](nat_prime_omega(a)) * alternating_sign[Int](nat_prime_omega(b))
        alternating_sign[Int](nat_prime_omega(a * b)) =
            alternating_sign[Int](nat_prime_omega(a)) * alternating_sign[Int](nat_prime_omega(b))
        nat_liouville(a * b) = alternating_sign[Int](nat_prime_omega(a * b))
        nat_liouville(a * b) =
            alternating_sign[Int](nat_prime_omega(a)) * alternating_sign[Int](nat_prime_omega(b))
        nat_liouville(a) = alternating_sign[Int](nat_prime_omega(a))
        nat_liouville(b) = alternating_sign[Int](nat_prime_omega(b))
        nat_liouville(a * b) = nat_liouville(a) * nat_liouville(b)
    }
}

/// Two does not divide three.
theorem nat_two_not_divides_three {
    not Nat.2.divides(Nat.3)
} by {
    if Nat.2.divides(Nat.3) {
        Nat.2 * Nat.1 = Nat.2
        exists(c: Nat) { Nat.2 * c = Nat.2 }
        Nat.2.divides(Nat.2)
        divides_suc_pair_imp_one(Nat.2, Nat.2)
        Nat.2 = Nat.1
        Nat.2 != Nat.1
        false
    }
}

/// A natural strictly between one and three is two.
theorem nat_between_one_and_three_is_two(k: Nat) {
    Nat.1 < k and k < Nat.3 implies k = Nat.2
} by {
    if Nat.1 < k and k < Nat.3 {
        lt_suc_right(k, Nat.2)
        k = Nat.2 or k < Nat.2
        if k < Nat.2 {
            lt_imp_lte_suc(Nat.1, k)
            Nat.2 <= k
            lte_and_lt(Nat.2, k, Nat.2)
            Nat.2 < Nat.2
            lt_not_ref(Nat.2)
            false
        } else {
            k = Nat.2
        }
    }
}

/// Three is prime.
theorem nat_three_prime {
    Nat.3.is_prime
} by {
    Nat.1 < Nat.2
    lt_suc(Nat.2)
    Nat.2 < Nat.3
    lt_trans(Nat.1, Nat.2, Nat.3)
    Nat.1 < Nat.3
    forall(k: Nat) {
        if Nat.1 < k and k < Nat.3 {
            nat_between_one_and_three_is_two(k)
            k = Nat.2
            nat_two_not_divides_three
            not Nat.2.divides(Nat.3)
            not k.divides(Nat.3)
        }
    }
    no_proper_divisor_imp_prime(Nat.3)
}

/// `lambda(3) = -1`.
theorem nat_liouville_three {
    nat_liouville(Nat.3) = -Int.1
} by {
    nat_three_prime
    Nat.3.is_prime
    prime_factorisation_prime(Nat.3)
    prime_factorisation(Nat.3) = List.singleton(Nat.3)
    List.singleton(Nat.3).length = Nat.1
    alternating_sign_suc[Int](Nat.0)
    alternating_sign[Int](Nat.0.suc) = -alternating_sign[Int](Nat.0)
    alternating_sign_zero[Int]
    alternating_sign[Int](Nat.0) = Int.1
    Nat.0.suc = Nat.1
    alternating_sign[Int](Nat.1) = -Int.1
    nat_prime_omega(Nat.3) = prime_factorisation(Nat.3).length
    nat_prime_omega(Nat.3) = Nat.1
    alternating_sign[Int](nat_prime_omega(Nat.3)) = alternating_sign[Int](Nat.1)
    alternating_sign[Int](nat_prime_omega(Nat.3)) = -Int.1
    nat_liouville(Nat.3) = alternating_sign[Int](nat_prime_omega(Nat.3))
    nat_liouville(Nat.3) = -Int.1
}

/// `lambda(4) = 1`, since four has two prime factors.
theorem nat_liouville_four {
    nat_liouville(Nat.4) = Int.1
} by {
    Nat.2 != Nat.0
    nat_liouville_mul(Nat.2, Nat.2)
    nat_liouville(Nat.2 * Nat.2) = nat_liouville(Nat.2) * nat_liouville(Nat.2)
    Nat.2 * Nat.2 = Nat.4
    nat_liouville(Nat.4) = nat_liouville(Nat.2) * nat_liouville(Nat.2)
    nat_liouville_two
    nat_liouville(Nat.2) = -Int.1
    nat_liouville(Nat.2) * nat_liouville(Nat.2) = -Int.1 * -Int.1
    -Int.1 * -Int.1 = Int.1
    nat_liouville(Nat.4) = Int.1
}

/// `lambda(6) = 1`, since six has two prime factors.
theorem nat_liouville_six {
    nat_liouville(Nat.6) = Int.1
} by {
    Nat.2 != Nat.0
    Nat.3 != Nat.0
    nat_liouville_mul(Nat.2, Nat.3)
    nat_liouville(Nat.2 * Nat.3) = nat_liouville(Nat.2) * nat_liouville(Nat.3)
    Nat.2 * Nat.3 = Nat.6
    nat_liouville(Nat.6) = nat_liouville(Nat.2) * nat_liouville(Nat.3)
    nat_liouville_two
    nat_liouville(Nat.2) = -Int.1
    nat_liouville_three
    nat_liouville(Nat.3) = -Int.1
    nat_liouville(Nat.2) * nat_liouville(Nat.3) = -Int.1 * -Int.1
    -Int.1 * -Int.1 = Int.1
    nat_liouville(Nat.6) = Int.1
}

/// `lambda(8) = -1`, since eight has three prime factors.
theorem nat_liouville_eight {
    nat_liouville(Nat.8) = -Int.1
} by {
    Nat.2 != Nat.0
    Nat.4 != Nat.0
    nat_liouville_mul(Nat.2, Nat.4)
    nat_liouville(Nat.2 * Nat.4) = nat_liouville(Nat.2) * nat_liouville(Nat.4)
    Nat.2 * Nat.4 = Nat.8
    nat_liouville(Nat.8) = nat_liouville(Nat.2) * nat_liouville(Nat.4)
    nat_liouville_two
    nat_liouville(Nat.2) = -Int.1
    nat_liouville_four
    nat_liouville(Nat.4) = Int.1
    -Int.1 * Int.1 = -Int.1
    nat_liouville(Nat.8) = -Int.1
}

// ---------------------------------------------------------------------------
// Divisor-sum identity:  sum_{d | n} lambda(d) = 1 if n is a perfect square,
// and 0 otherwise.
//
// A full proof needs the prime-power decomposition of the divisor sum; the
// general statement is therefore recorded here and proved below for small n.
//
// theorem nat_liouville_divisor_sum(n: Nat) {
//     Nat.0 < n implies
//         divisor_sum_int_fn(nat_liouville)(n) = nat_liouville_divisor_sum_value(n)
// } by {
//     // Deferred: the divisor sum factors over the prime powers of n, and each
//     // prime-power sum 1 + (-1) + ... + (-1)^e vanishes unless e is even.
// }
// ---------------------------------------------------------------------------

/// The divisor sum of an integer-valued function over the positive divisors of `n`.
define divisor_sum_int_fn(f: Nat -> Int) -> (Nat -> Int) {
    function(n: Nat) { sum(map(divisor_list(n), f)) }
}

/// The Liouville divisor-sum value: one at perfect squares, zero elsewhere.
define nat_liouville_divisor_sum_value(n: Nat) -> Int {
    if is_square(n) { Int.1 } else { Int.0 }
}

/// Two divides four.
theorem nat_two_divides_four {
    Nat.2.divides(Nat.4)
} by {
    Nat.2 * Nat.2 = Nat.4
    exists(c: Nat) { Nat.2 * c = Nat.4 }
    Nat.2.divides(Nat.4)
}

/// Three does not divide four.
theorem nat_three_not_divides_four {
    not Nat.3.divides(Nat.4)
} by {
    if Nat.3.divides(Nat.4) {
        Nat.3 * Nat.1 = Nat.3
        exists(c: Nat) { Nat.3 * c = Nat.3 }
        Nat.3.divides(Nat.3)
        divides_suc_pair_imp_one(Nat.3, Nat.3)
        Nat.3 = Nat.1
        Nat.3 != Nat.1
        false
    }
}

/// The divisor list of four is `[4, 2, 1]`.
theorem divisor_list_four {
    divisor_list(Nat.4) = List.cons(Nat.4, List.cons(Nat.2, List.cons(Nat.1, List.nil[Nat])))
} by {
    divides_self(Nat.4)
    Nat.4.divides(Nat.4)
    divisors_up_to_suc_yes(Nat.4, Nat.3)
    divisors_up_to(Nat.4, Nat.3.suc) = List.cons(Nat.3.suc, divisors_up_to(Nat.4, Nat.3))
    Nat.3.suc = Nat.4
    divisors_up_to(Nat.4, Nat.4) = List.cons(Nat.4, divisors_up_to(Nat.4, Nat.3))
    nat_three_not_divides_four
    not Nat.3.divides(Nat.4)
    divisors_up_to_suc_no(Nat.4, Nat.2)
    divisors_up_to(Nat.4, Nat.2.suc) = divisors_up_to(Nat.4, Nat.2)
    Nat.2.suc = Nat.3
    divisors_up_to(Nat.4, Nat.3) = divisors_up_to(Nat.4, Nat.2)
    nat_two_divides_four
    Nat.2.divides(Nat.4)
    divisors_up_to_suc_yes(Nat.4, Nat.1)
    divisors_up_to(Nat.4, Nat.1.suc) = List.cons(Nat.1.suc, divisors_up_to(Nat.4, Nat.1))
    Nat.1.suc = Nat.2
    divisors_up_to(Nat.4, Nat.2) = List.cons(Nat.2, divisors_up_to(Nat.4, Nat.1))
    divisors_up_to_one(Nat.4)
    divisors_up_to(Nat.4, Nat.1) = List.cons(Nat.1, List.nil[Nat])
    divisors_up_to(Nat.4, Nat.4) =
        List.cons(Nat.4, List.cons(Nat.2, List.cons(Nat.1, List.nil[Nat])))
    divisor_list(Nat.4) = divisors_up_to(Nat.4, Nat.4)
}

/// Two divides six.
theorem nat_two_divides_six {
    Nat.2.divides(Nat.6)
} by {
    Nat.2 * Nat.3 = Nat.6
    exists(c: Nat) { Nat.2 * c = Nat.6 }
    Nat.2.divides(Nat.6)
}

/// Three divides six.
theorem nat_three_divides_six {
    Nat.3.divides(Nat.6)
} by {
    Nat.3 * Nat.2 = Nat.6
    exists(c: Nat) { Nat.3 * c = Nat.6 }
    Nat.3.divides(Nat.6)
}

/// Four does not divide six.
theorem nat_four_not_divides_six {
    not Nat.4.divides(Nat.6)
} by {
    if Nat.4.divides(Nat.6) {
        let c: Nat satisfy { Nat.4 * c = Nat.6 }
        Nat.4 * c = Nat.6
        Nat.4 * c = Nat.2 * (Nat.2 * c)
        Nat.2 * (Nat.2 * c) = Nat.6
        Nat.2 * Nat.3 = Nat.6
        mul_cancel_left(Nat.2, Nat.2 * c, Nat.3)
        Nat.2 * c = Nat.3
        exists(x: Nat) { Nat.2 * x = Nat.3 }
        Nat.2.divides(Nat.3)
        nat_two_not_divides_three
        false
    }
}

/// Five does not divide six.
theorem nat_five_not_divides_six {
    not Nat.5.divides(Nat.6)
} by {
    if Nat.5.divides(Nat.6) {
        Nat.5 * Nat.1 = Nat.5
        exists(c: Nat) { Nat.5 * c = Nat.5 }
        Nat.5.divides(Nat.5)
        divides_suc_pair_imp_one(Nat.5, Nat.5)
        Nat.5 = Nat.1
        Nat.5 != Nat.1
        false
    }
}

/// The divisor list of six is `[6, 3, 2, 1]`.
theorem divisor_list_six {
    divisor_list(Nat.6) =
        List.cons(Nat.6, List.cons(Nat.3, List.cons(Nat.2, List.cons(Nat.1, List.nil[Nat]))))
} by {
    divides_self(Nat.6)
    Nat.6.divides(Nat.6)
    divisors_up_to_suc_yes(Nat.6, Nat.5)
    divisors_up_to(Nat.6, Nat.5.suc) = List.cons(Nat.5.suc, divisors_up_to(Nat.6, Nat.5))
    Nat.5.suc = Nat.6
    divisors_up_to(Nat.6, Nat.6) = List.cons(Nat.6, divisors_up_to(Nat.6, Nat.5))
    nat_five_not_divides_six
    not Nat.5.divides(Nat.6)
    divisors_up_to_suc_no(Nat.6, Nat.4)
    divisors_up_to(Nat.6, Nat.4.suc) = divisors_up_to(Nat.6, Nat.4)
    Nat.4.suc = Nat.5
    divisors_up_to(Nat.6, Nat.5) = divisors_up_to(Nat.6, Nat.4)
    nat_four_not_divides_six
    not Nat.4.divides(Nat.6)
    divisors_up_to_suc_no(Nat.6, Nat.3)
    divisors_up_to(Nat.6, Nat.3.suc) = divisors_up_to(Nat.6, Nat.3)
    Nat.3.suc = Nat.4
    divisors_up_to(Nat.6, Nat.4) = divisors_up_to(Nat.6, Nat.3)
    nat_three_divides_six
    Nat.3.divides(Nat.6)
    divisors_up_to_suc_yes(Nat.6, Nat.2)
    divisors_up_to(Nat.6, Nat.2.suc) = List.cons(Nat.2.suc, divisors_up_to(Nat.6, Nat.2))
    Nat.2.suc = Nat.3
    divisors_up_to(Nat.6, Nat.3) = List.cons(Nat.3, divisors_up_to(Nat.6, Nat.2))
    nat_two_divides_six
    Nat.2.divides(Nat.6)
    divisors_up_to_suc_yes(Nat.6, Nat.1)
    divisors_up_to(Nat.6, Nat.1.suc) = List.cons(Nat.1.suc, divisors_up_to(Nat.6, Nat.1))
    Nat.1.suc = Nat.2
    divisors_up_to(Nat.6, Nat.2) = List.cons(Nat.2, divisors_up_to(Nat.6, Nat.1))
    divisors_up_to_one(Nat.6)
    divisors_up_to(Nat.6, Nat.1) = List.cons(Nat.1, List.nil[Nat])
    divisors_up_to(Nat.6, Nat.6) =
        List.cons(Nat.6, List.cons(Nat.3, List.cons(Nat.2, List.cons(Nat.1, List.nil[Nat]))))
    divisor_list(Nat.6) = divisors_up_to(Nat.6, Nat.6)
}

/// Four is a perfect square.
theorem nat_four_is_square {
    is_square(Nat.4)
} by {
    Nat.4 = Nat.2 * Nat.2
    is_square_intro(Nat.4, Nat.2)
    is_square(Nat.4)
}

/// Two is not a perfect square: a square has an even number of prime factors,
/// so its Liouville value is one, unlike two.
theorem nat_two_not_square {
    not is_square(Nat.2)
} by {
    if is_square(Nat.2) {
        is_square_witness(Nat.2)
        let k: Nat satisfy { Nat.2 = k * k }
        Nat.2 = k * k
        k * k = Nat.2
        if k = Nat.0 {
            k * k = Nat.0
            Nat.2 = Nat.0
            false
        }
        k != Nat.0
        nat_liouville_mul(k, k)
        nat_liouville(k * k) = nat_liouville(k) * nat_liouville(k)
        nat_liouville(Nat.2) = nat_liouville(k) * nat_liouville(k)
        nat_liouville_one_or_neg_one(k)
        nat_liouville(k) = Int.1 or nat_liouville(k) = -Int.1
        if nat_liouville(k) = Int.1 {
            nat_liouville(k) * nat_liouville(k) = Int.1 * Int.1
            Int.1 * Int.1 = Int.1
            nat_liouville(k) * nat_liouville(k) = Int.1
        } else {
            nat_liouville(k) = -Int.1
            nat_liouville(k) * nat_liouville(k) = -Int.1 * -Int.1
            -Int.1 * -Int.1 = Int.1
            nat_liouville(k) * nat_liouville(k) = Int.1
        }
        nat_liouville(k) * nat_liouville(k) = Int.1
        nat_liouville(Nat.2) = Int.1
        nat_liouville_two
        nat_liouville(Nat.2) = -Int.1
        Int.1 = -Int.1
        false
    }
}

/// Six is not a perfect square.
theorem nat_six_not_square {
    not is_square(Nat.6)
} by {
    if is_square(Nat.6) {
        is_square_witness(Nat.6)
        let k: Nat satisfy { Nat.6 = k * k }
        Nat.6 = k * k
        k * k = Nat.6
        if k = Nat.0 {
            k * k = Nat.0
            Nat.6 = Nat.0
            false
        }
        k != Nat.0
        Nat.0 < k
        Nat.0 < Nat.6
        exists(c: Nat) { k * c = Nat.6 }
        k.divides(Nat.6)
        divisor_list_contains_of(Nat.6, k)
        divisor_list(Nat.6).contains(k)
        divisor_list_six
        divisor_list(Nat.6) =
            List.cons(Nat.6, List.cons(Nat.3, List.cons(Nat.2, List.cons(Nat.1, List.nil[Nat]))))
        if k = Nat.6 {
            k * k = Nat.6 * Nat.6
            Nat.6 * Nat.6 = Nat.36
            k * k = Nat.36
            Nat.6 = Nat.36
            Nat.6 = Nat.6 * Nat.6
            Nat.6 * Nat.1 = Nat.6 * Nat.6
            mul_cancel_left(Nat.6, Nat.1, Nat.6)
            Nat.1 = Nat.6
            Nat.1 != Nat.6
            false
        } else {
            if k = Nat.3 {
                k * k = Nat.3 * Nat.3
                Nat.3 * Nat.3 = Nat.9
                k * k = Nat.9
                Nat.6 = Nat.9
                false
            } else {
                if k = Nat.2 {
                    k * k = Nat.2 * Nat.2
                    Nat.2 * Nat.2 = Nat.4
                    k * k = Nat.4
                    Nat.6 = Nat.4
                    false
                } else {
                    if k = Nat.1 {
                        k * k = Nat.1 * Nat.1
                        Nat.1 * Nat.1 = Nat.1
                        k * k = Nat.1
                        Nat.6 = Nat.1
                        false
                    } else {
                        List.cons(Nat.6,
                            List.cons(Nat.3, List.cons(Nat.2, List.cons(Nat.1, List.nil[Nat])))).contains(k)
                        List.cons(Nat.3, List.cons(Nat.2, List.cons(Nat.1, List.nil[Nat]))).contains(k)
                        List.cons(Nat.2, List.cons(Nat.1, List.nil[Nat])).contains(k)
                        List.cons(Nat.1, List.nil[Nat]).contains(k)
                        List.nil[Nat].contains(k)
                        false
                    }
                }
            }
        }
        false
    }
}

/// The Liouville divisor sum at one is one.
theorem nat_liouville_divisor_sum_one {
    divisor_sum_int_fn(nat_liouville)(Nat.1) = Int.1
} by {
    divisor_sum_int_fn(nat_liouville)(Nat.1) = sum(map(divisor_list(Nat.1), nat_liouville))
    divisor_list_one
    divisor_list(Nat.1) = List.cons(Nat.1, List.nil[Nat])
    map(List.cons(Nat.1, List.nil[Nat]), nat_liouville) =
        List.cons(nat_liouville(Nat.1), map(List.nil[Nat], nat_liouville))
    map(List.nil[Nat], nat_liouville) = List.nil[Int]
    sum(List.cons(nat_liouville(Nat.1), List.nil[Int])) =
        nat_liouville(Nat.1) + sum(List.nil[Int])
    sum(List.nil[Int]) = Int.0
    nat_liouville(Nat.1) + Int.0 = nat_liouville(Nat.1)
    nat_liouville_one
    nat_liouville(Nat.1) = Int.1
    sum(List.cons(nat_liouville(Nat.1), List.nil[Int])) = Int.1
    sum(map(divisor_list(Nat.1), nat_liouville)) = Int.1
    divisor_sum_int_fn(nat_liouville)(Nat.1) = Int.1
}

/// The Liouville divisor sum at two is zero.
theorem nat_liouville_divisor_sum_two {
    divisor_sum_int_fn(nat_liouville)(Nat.2) = Int.0
} by {
    divisor_sum_int_fn(nat_liouville)(Nat.2) = sum(map(divisor_list(Nat.2), nat_liouville))
    nat_two_prime_local
    divisor_list_prime(Nat.2)
    divisor_list(Nat.2) = List.cons(Nat.2, List.cons(Nat.1, List.nil[Nat]))
    map(List.cons(Nat.2, List.cons(Nat.1, List.nil[Nat])), nat_liouville) =
        List.cons(nat_liouville(Nat.2), map(List.cons(Nat.1, List.nil[Nat]), nat_liouville))
    map(List.cons(Nat.1, List.nil[Nat]), nat_liouville) =
        List.cons(nat_liouville(Nat.1), map(List.nil[Nat], nat_liouville))
    map(List.nil[Nat], nat_liouville) = List.nil[Int]
    sum(List.cons(nat_liouville(Nat.2), List.cons(nat_liouville(Nat.1), List.nil[Int]))) =
        nat_liouville(Nat.2) + sum(List.cons(nat_liouville(Nat.1), List.nil[Int]))
    sum(List.cons(nat_liouville(Nat.1), List.nil[Int])) =
        nat_liouville(Nat.1) + sum(List.nil[Int])
    sum(List.nil[Int]) = Int.0
    nat_liouville(Nat.1) + Int.0 = nat_liouville(Nat.1)
    nat_liouville_one
    nat_liouville(Nat.1) = Int.1
    sum(List.cons(nat_liouville(Nat.1), List.nil[Int])) = Int.1
    nat_liouville_two
    nat_liouville(Nat.2) = -Int.1
    nat_liouville(Nat.2) + Int.1 = -Int.1 + Int.1
    -Int.1 + Int.1 = Int.0
    sum(List.cons(nat_liouville(Nat.2), List.cons(nat_liouville(Nat.1), List.nil[Int]))) = Int.0
    sum(map(divisor_list(Nat.2), nat_liouville)) = Int.0
    divisor_sum_int_fn(nat_liouville)(Nat.2) = Int.0
}

/// The Liouville divisor sum at four is one.
theorem nat_liouville_divisor_sum_four {
    divisor_sum_int_fn(nat_liouville)(Nat.4) = Int.1
} by {
    divisor_sum_int_fn(nat_liouville)(Nat.4) = sum(map(divisor_list(Nat.4), nat_liouville))
    divisor_list_four
    divisor_list(Nat.4) = List.cons(Nat.4, List.cons(Nat.2, List.cons(Nat.1, List.nil[Nat])))
    map(List.cons(Nat.4, List.cons(Nat.2, List.cons(Nat.1, List.nil[Nat]))), nat_liouville) =
        List.cons(nat_liouville(Nat.4),
            map(List.cons(Nat.2, List.cons(Nat.1, List.nil[Nat])), nat_liouville))
    map(List.cons(Nat.2, List.cons(Nat.1, List.nil[Nat])), nat_liouville) =
        List.cons(nat_liouville(Nat.2), map(List.cons(Nat.1, List.nil[Nat]), nat_liouville))
    map(List.cons(Nat.1, List.nil[Nat]), nat_liouville) =
        List.cons(nat_liouville(Nat.1), map(List.nil[Nat], nat_liouville))
    map(List.nil[Nat], nat_liouville) = List.nil[Int]
    sum(List.cons(nat_liouville(Nat.4),
        List.cons(nat_liouville(Nat.2), List.cons(nat_liouville(Nat.1), List.nil[Int])))) =
        nat_liouville(Nat.4) +
            sum(List.cons(nat_liouville(Nat.2), List.cons(nat_liouville(Nat.1), List.nil[Int])))
    sum(List.cons(nat_liouville(Nat.2), List.cons(nat_liouville(Nat.1), List.nil[Int]))) =
        nat_liouville(Nat.2) + sum(List.cons(nat_liouville(Nat.1), List.nil[Int]))
    sum(List.cons(nat_liouville(Nat.1), List.nil[Int])) =
        nat_liouville(Nat.1) + sum(List.nil[Int])
    sum(List.nil[Int]) = Int.0
    nat_liouville(Nat.1) + Int.0 = nat_liouville(Nat.1)
    nat_liouville_one
    nat_liouville(Nat.1) = Int.1
    sum(List.cons(nat_liouville(Nat.1), List.nil[Int])) = Int.1
    nat_liouville_two
    nat_liouville(Nat.2) = -Int.1
    nat_liouville(Nat.2) + Int.1 = -Int.1 + Int.1
    -Int.1 + Int.1 = Int.0
    sum(List.cons(nat_liouville(Nat.2), List.cons(nat_liouville(Nat.1), List.nil[Int]))) = Int.0
    nat_liouville_four
    nat_liouville(Nat.4) = Int.1
    nat_liouville(Nat.4) + Int.0 = Int.1 + Int.0
    Int.1 + Int.0 = Int.1
    sum(map(divisor_list(Nat.4), nat_liouville)) = Int.1
    divisor_sum_int_fn(nat_liouville)(Nat.4) = Int.1
}

/// The Liouville divisor sum at six is zero.
theorem nat_liouville_divisor_sum_six {
    divisor_sum_int_fn(nat_liouville)(Nat.6) = Int.0
} by {
    divisor_sum_int_fn(nat_liouville)(Nat.6) = sum(map(divisor_list(Nat.6), nat_liouville))
    divisor_list_six
    divisor_list(Nat.6) =
        List.cons(Nat.6, List.cons(Nat.3, List.cons(Nat.2, List.cons(Nat.1, List.nil[Nat]))))
    map(List.cons(Nat.6, List.cons(Nat.3, List.cons(Nat.2, List.cons(Nat.1, List.nil[Nat])))), nat_liouville) =
        List.cons(nat_liouville(Nat.6),
            map(List.cons(Nat.3, List.cons(Nat.2, List.cons(Nat.1, List.nil[Nat]))), nat_liouville))
    map(List.cons(Nat.3, List.cons(Nat.2, List.cons(Nat.1, List.nil[Nat]))), nat_liouville) =
        List.cons(nat_liouville(Nat.3),
            map(List.cons(Nat.2, List.cons(Nat.1, List.nil[Nat])), nat_liouville))
    map(List.cons(Nat.2, List.cons(Nat.1, List.nil[Nat])), nat_liouville) =
        List.cons(nat_liouville(Nat.2), map(List.cons(Nat.1, List.nil[Nat]), nat_liouville))
    map(List.cons(Nat.1, List.nil[Nat]), nat_liouville) =
        List.cons(nat_liouville(Nat.1), map(List.nil[Nat], nat_liouville))
    map(List.nil[Nat], nat_liouville) = List.nil[Int]
    sum(List.cons(nat_liouville(Nat.6),
        List.cons(nat_liouville(Nat.3),
            List.cons(nat_liouville(Nat.2), List.cons(nat_liouville(Nat.1), List.nil[Int]))))) =
        nat_liouville(Nat.6) +
            sum(List.cons(nat_liouville(Nat.3),
                List.cons(nat_liouville(Nat.2), List.cons(nat_liouville(Nat.1), List.nil[Int]))))
    sum(List.cons(nat_liouville(Nat.3),
        List.cons(nat_liouville(Nat.2), List.cons(nat_liouville(Nat.1), List.nil[Int])))) =
        nat_liouville(Nat.3) +
            sum(List.cons(nat_liouville(Nat.2), List.cons(nat_liouville(Nat.1), List.nil[Int])))
    sum(List.cons(nat_liouville(Nat.2), List.cons(nat_liouville(Nat.1), List.nil[Int]))) =
        nat_liouville(Nat.2) + sum(List.cons(nat_liouville(Nat.1), List.nil[Int]))
    sum(List.cons(nat_liouville(Nat.1), List.nil[Int])) =
        nat_liouville(Nat.1) + sum(List.nil[Int])
    sum(List.nil[Int]) = Int.0
    nat_liouville(Nat.1) + Int.0 = nat_liouville(Nat.1)
    nat_liouville_one
    nat_liouville(Nat.1) = Int.1
    sum(List.cons(nat_liouville(Nat.1), List.nil[Int])) = Int.1
    nat_liouville_two
    nat_liouville(Nat.2) = -Int.1
    nat_liouville(Nat.2) + Int.1 = -Int.1 + Int.1
    -Int.1 + Int.1 = Int.0
    sum(List.cons(nat_liouville(Nat.2), List.cons(nat_liouville(Nat.1), List.nil[Int]))) = Int.0
    nat_liouville_three
    nat_liouville(Nat.3) = -Int.1
    nat_liouville(Nat.3) + Int.0 = -Int.1 + Int.0
    -Int.1 + Int.0 = -Int.1
    sum(List.cons(nat_liouville(Nat.3),
        List.cons(nat_liouville(Nat.2), List.cons(nat_liouville(Nat.1), List.nil[Int])))) = -Int.1
    nat_liouville_six
    nat_liouville(Nat.6) = Int.1
    nat_liouville(Nat.6) + -Int.1 = Int.1 + -Int.1
    Int.1 + -Int.1 = Int.0
    sum(map(divisor_list(Nat.6), nat_liouville)) = Int.0
    divisor_sum_int_fn(nat_liouville)(Nat.6) = Int.0
}

/// The Liouville divisor-sum value at one is one, matching the sum.
theorem nat_liouville_divisor_sum_value_one {
    nat_liouville_divisor_sum_value(Nat.1) = Int.1
} by {
    one_is_square
    is_square(Nat.1)
    nat_liouville_divisor_sum_value(Nat.1) = Int.1
}

/// The Liouville divisor-sum value at two is zero, matching the sum.
theorem nat_liouville_divisor_sum_value_two {
    nat_liouville_divisor_sum_value(Nat.2) = Int.0
} by {
    nat_two_not_square
    not is_square(Nat.2)
    nat_liouville_divisor_sum_value(Nat.2) = Int.0
}

/// The Liouville divisor-sum value at four is one, matching the sum.
theorem nat_liouville_divisor_sum_value_four {
    nat_liouville_divisor_sum_value(Nat.4) = Int.1
} by {
    nat_four_is_square
    is_square(Nat.4)
    nat_liouville_divisor_sum_value(Nat.4) = Int.1
}

/// The Liouville divisor-sum value at six is zero, matching the sum.
theorem nat_liouville_divisor_sum_value_six {
    nat_liouville_divisor_sum_value(Nat.6) = Int.0
} by {
    nat_six_not_square
    not is_square(Nat.6)
    nat_liouville_divisor_sum_value(Nat.6) = Int.0
}

/// The divisor-sum identity for the Liouville function at the small values:
/// `sum_{d | n} lambda(d) = 1` if `n` is a perfect square and `0` otherwise,
/// for `n` in `{1, 2, 4, 6}`.
theorem nat_liouville_divisor_sum_small(n: Nat) {
    (n = Nat.1 or n = Nat.2 or n = Nat.4 or n = Nat.6) implies
        divisor_sum_int_fn(nat_liouville)(n) = nat_liouville_divisor_sum_value(n)
} by {
    if n = Nat.1 or n = Nat.2 or n = Nat.4 or n = Nat.6 {
        if n = Nat.1 {
            nat_liouville_divisor_sum_one
            nat_liouville_divisor_sum_value(Nat.1) = Int.1
            divisor_sum_int_fn(nat_liouville)(n) = nat_liouville_divisor_sum_value(n)
        } else {
            if n = Nat.2 {
                nat_liouville_divisor_sum_two
                nat_liouville_divisor_sum_value_two
                divisor_sum_int_fn(nat_liouville)(n) = nat_liouville_divisor_sum_value(n)
            } else {
                if n = Nat.4 {
                    nat_liouville_divisor_sum_four
                    nat_liouville_divisor_sum_value_four
                    divisor_sum_int_fn(nat_liouville)(n) = nat_liouville_divisor_sum_value(n)
                } else {
                    n = Nat.6
                    nat_liouville_divisor_sum_six
                    nat_liouville_divisor_sum_value_six
                    divisor_sum_int_fn(nat_liouville)(n) = nat_liouville_divisor_sum_value(n)
                }
            }
        }
    }
}
