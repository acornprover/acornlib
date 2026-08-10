/// Approximation theorems: Dirichlet's approximation theorem and the deeper
/// classical results on approximating irrational numbers.
///
/// The library's continued-fraction machinery (continued_fraction.ac,
/// continued_fraction_convergents.ac, continued_fraction_approx.ac) builds,
/// for every coefficient sequence with positive tail, the real limit `alpha`
/// of the convergents `p_n / q_n` together with the approximation estimate
/// `|alpha - p_n / q_n| < 1 / q_n^2` and Dirichlet's theorem
/// (diophantine_approx.ac, approximation_deep.ac).  This file collects the
/// classical approximation theorems:
///
///   1. Dirichlet's theorem, restated in the three forms the library proves
///      (Section 1): the per-convergent estimate `|alpha - p_n / q_n| <
///      1 / q_n^2`, infinitely many good approximations with unbounded
///      denominators, and the box-principle form `|q * alpha - p| < 1 / bound`
///      with `1 <= q <= bound`.  The fully general statement for an arbitrary
///      irrational real is recorded as a comment (Section 2), since the
///      library does not yet construct the continued-fraction expansion of an
///      arbitrary real number.
///
///   2. The Hurwitz theorem — `|alpha - p / q| < 1 / (sqrt(5) * q^2)` for
///      infinitely many `p / q`, with `sqrt(5)` the best possible constant —
///      is stated with its proof outline (Section 3).  The golden-ratio
///      constant `1 / sqrt(5)` is optimal: for the golden ratio
///      `(1 + sqrt(5)) / 2` the bound cannot be improved.
///
///   3. Liouville's theorem — an algebraic number of degree `n` is
///      approximable by rationals only to order `n`, i.e. there is a constant
///      `c > 0` with `|alpha - p / q| > c / q^n` for all but finitely many
///      `p / q` — is stated with its proof outline (Section 4).
///
///   4. The irrationality of `e` (Section 5), restated from
///      approximation_deep.ac, where the classical Fourier proof is outlined;
///      the statement is not yet proved there.
///
///   5. The equidistribution statement — the fractional parts of `n * alpha`
///      are dense in `[0, 1]` for irrational `alpha` — is stated with its
///      proof outline (Section 6).
///
/// The deliverable targets of the surrounding development are (a) the
/// restatement of Dirichlet's theorem from diophantine_approx.ac, proved in
/// Section 1, and (d) the irrationality of `e`, stated in Section 5 in the
/// form recorded (unproved) in approximation_deep.ac; the deeper statements
/// (Sections 3, 4, 6) are recorded for future work.
from nat import Nat
from rat import Rat
from real import Real
from number_theory.continued_fraction_convergents import positive_continued_fraction_sequence_tail,
    continued_fraction_convergent_denominator
from number_theory.continued_fraction_approx import continued_fraction_real_limit,
    continued_fraction_real_convergent_value
from number_theory.diophantine_approx import diophantine_approximation_estimate,
    diophantine_good_approximations_unbounded, diophantine_dirichlet,
    dirichlet_good_approximation

numerals Nat

// ============================================================================
// Section 1: Dirichlet's approximation theorem, restated
// ============================================================================

/// Dirichlet's approximation estimate: the continued-fraction limit is within
/// `1 / q_n^2` of the `n`-th convergent.
///
/// This restates `diophantine_approximation_estimate` from
/// diophantine_approx.ac; the full proof lives there.
theorem approximation_theorems_dirichlet_estimate(
    coefficients: Nat -> Nat, n: Nat
) {
    positive_continued_fraction_sequence_tail(coefficients) implies (continued_fraction_real_limit(coefficients) - continued_fraction_real_convergent_value(coefficients, n)).abs < Real.from_rat(Rat.1 / Rat.from_nat(
            continued_fraction_convergent_denominator(coefficients, n) *
            continued_fraction_convergent_denominator(coefficients, n)))
} by {
    if positive_continued_fraction_sequence_tail(coefficients) {
        diophantine_approximation_estimate(coefficients, n)
        (continued_fraction_real_limit(coefficients) - continued_fraction_real_convergent_value(coefficients, n)).abs < Real.from_rat(Rat.1 / Rat.from_nat(
                continued_fraction_convergent_denominator(coefficients, n) *
                continued_fraction_convergent_denominator(coefficients, n)))
    }
}

/// Dirichlet's theorem: the continued-fraction limit has infinitely many good
/// rational approximations.  Beyond every bound `n` there is a convergent
/// `p / q` with denominator exceeding `n` and `|alpha - p / q| < 1 / q^2`.
///
/// This restates `diophantine_good_approximations_unbounded` from
/// diophantine_approx.ac (and `approximation_deep_infinitely_many_good_
/// approximations` from approximation_deep.ac); the proof lives there.  For
/// every irrational `alpha` the continued-fraction expansion is infinite (the
/// expansion of a rational number terminates), so this is the library's form
/// of the classical statement that every irrational number has infinitely
/// many `1 / q^2`-approximations.
theorem approximation_theorems_dirichlet_infinitely_many(
    coefficients: Nat -> Nat, n: Nat
) {
    positive_continued_fraction_sequence_tail(coefficients) implies
        exists(p: Nat, q: Nat) {
            n < q and
            (continued_fraction_real_limit(coefficients) -
                Real.from_rat(Rat.from_nat(p) / Rat.from_nat(q))).abs <
                Real.from_rat(Rat.1 / Rat.from_nat(q * q))
        }
} by {
    if positive_continued_fraction_sequence_tail(coefficients) {
        diophantine_good_approximations_unbounded(coefficients, n)
        exists(p: Nat, q: Nat) {
            n < q and
            (continued_fraction_real_limit(coefficients) -
                Real.from_rat(Rat.from_nat(p) / Rat.from_nat(q))).abs <
                Real.from_rat(Rat.1 / Rat.from_nat(q * q))
        }
    }
}

/// Dirichlet's approximation theorem in the box-principle form: for the
/// continued-fraction limit and every bound >= 1 there exist naturals p, q
/// with `1 <= q <= bound` and `|q * alpha - p| < 1 / bound`.
///
/// This restates `diophantine_dirichlet` from diophantine_approx.ac; the
/// proof lives there.
theorem approximation_theorems_dirichlet_box(
    coefficients: Nat -> Nat, bound: Nat
) {
    positive_continued_fraction_sequence_tail(coefficients) and Nat.1 <= bound implies
    exists(p: Nat, q: Nat) {
        dirichlet_good_approximation(continued_fraction_real_limit(coefficients), p, q, bound)
    }
} by {
    if positive_continued_fraction_sequence_tail(coefficients) and Nat.1 <= bound {
        diophantine_dirichlet(coefficients, bound)
        exists(p: Nat, q: Nat) {
            dirichlet_good_approximation(continued_fraction_real_limit(coefficients), p, q, bound)
        }
    }
}

// ============================================================================
// Section 2: Dirichlet's theorem for an arbitrary irrational real
// ============================================================================

// Dirichlet's approximation theorem for an arbitrary real alpha: every
// irrational alpha has infinitely many rational approximations p / q with
// |alpha - p / q| < 1 / q^2.  The library proves the convergent versions of
// Section 1 for every coefficient sequence with positive tail, and every
// irrational number has an infinite continued-fraction expansion, so the
// theorem below is the classical statement in the library's idiom.  It is
// not proved: the library does not yet construct the continued-fraction
// expansion of an arbitrary real number (there is no floor operation on
// `Real` from which to build the expansion), so the statement is recorded
// here for future work, exactly as in diophantine_approx.ac (Section 4).
//
// theorem approximation_theorems_dirichlet_general(alpha: Real) {
//     not exists(a: Nat, b: Nat) {
//         Nat.1 <= b and alpha = Real.from_rat(Rat.from_nat(a) / Rat.from_nat(b))
//     } implies forall(n: Nat) {
//         exists(p: Nat, q: Nat) {
//             n < q and
//             (alpha - Real.from_rat(Rat.from_nat(p) / Rat.from_nat(q))).abs <
//                 Real.from_rat(Rat.1 / Rat.from_nat(q * q))
//         }
//     }
// }

// ============================================================================
// Section 3: the Hurwitz theorem
// ============================================================================

// The Hurwitz theorem: for every irrational alpha there are infinitely many
// reduced p / q with
//
//     |alpha - p / q| < 1 / (sqrt(5) * q^2),
//
// and the constant sqrt(5) is the best possible: no larger constant works
// for every irrational (the golden ratio phi = (1 + sqrt(5)) / 2 attains the
// bound — every convergent of phi satisfies |phi - p_n / q_n| >= 1 / (sqrt(5)
// * q_n^2) asymptotically — so the constant in the theorem cannot be
// increased).
//
// Proof outline (from the continued-fraction expansion).  Every irrational
// alpha has an infinite continued fraction, and its convergents satisfy the
// gap estimate
//
//     |alpha - p_n / q_n| < 1 / (q_n * q_{n+1}),
//
// which the library proves in continued_fraction_approx.ac and restates in
// diophantine_approx.ac (diophantine_limit_gap_approx).  The denominators
// satisfy the recurrence q_{n+1} = a_{n+1} * q_n + q_{n-1}, where a_{n+1} is
// the (n + 1)-st partial quotient.  The classical argument shows that among
// any three consecutive convergents at least one satisfies
//
//     |alpha - p / q| < 1 / (sqrt(5) * q^2):
//
// if |alpha - p_k / q_k| >= 1 / (sqrt(5) * q_k^2) held for k = n, n + 1,
// n + 2, then the gap estimate forces q_{n+1} <= sqrt(5) * q_n and
// q_{n+2} <= sqrt(5) * q_{n+1}; combined with the recurrence
// q_{n+2} = a_{n+2} * q_{n+1} + q_n and the identity
// q_{n+1} = a_{n+1} * q_n + q_{n-1}, a short inequality chase shows the
// ratios q_{n+1} / q_n and q_{n+2} / q_{n+1} must both be at most the golden
// ratio phi = (1 + sqrt(5)) / 2, and then the determinant identity
// |p_n * q_{n+1} - p_{n+1} * q_n| = 1 yields a contradiction with the three
// lower bounds.  The golden ratio phi is the extremal case: its partial
// quotients are all one, the ratios q_{n+1} / q_n tend to phi, and the
// constant sqrt(5) is exactly what the inequality chase allows.
//
// The library's continued-fraction machinery proves the gap estimate, the
// recurrence for the denominators, and the determinant identity, but does
// not yet compare q_{n+1} against sqrt(5) * q_n through the partial
// quotients (the golden-ratio case analysis), and there is no arithmetic on
// square roots of naturals in `Real` exposed for the constant; the statement
// is recorded here for future work.
//
// theorem approximation_theorems_hurwitz(alpha: Real) {
//     not exists(a: Nat, b: Nat) {
//         Nat.1 <= b and alpha = Real.from_rat(Rat.from_nat(a) / Rat.from_nat(b))
//     } implies exists(s: Real) {
//         s * s = Real.from_rat(Rat.5) and s > Real.0 and
//         forall(n: Nat) {
//             exists(p: Nat, q: Nat) {
//                 n < q and
//                 (alpha - Real.from_rat(Rat.from_nat(p) / Rat.from_nat(q))).abs <
//                     Real.from_rat(Rat.1 / Rat.from_nat(q * q)) / s
//             }
//         }
//     }
// }

// ============================================================================
// Section 4: Liouville's theorem
// ============================================================================

// Liouville's theorem: if alpha is a real algebraic number of degree n >= 1,
// then there is a constant c > 0 such that
//
//     |alpha - p / q| > c / q^n
//
// for every rational p / q with q >= 1 — equivalently, for all but finitely
// many rationals p / q.
//
// Proof outline.  Let P be the minimal polynomial of alpha: P has integer
// coefficients, degree n, and P(alpha) = 0, and P is irreducible over the
// rationals.  Let M bound |P'| on the interval [alpha - 1, alpha + 1].  For a
// rational p / q in that interval, the mean value theorem gives
//
//     |P(p / q)| = |P(p / q) - P(alpha)| <= M * |p / q - alpha|.
//
// On the other hand, q^n * P(p / q) is a nonzero integer: clearing the
// denominators of P turns q^n * P(p / q) into an integer combination of
// powers of p and q, and it cannot vanish, because P is irreducible of
// degree n and p / q is rational of degree one (so P(p / q) != 0).  Hence
//
//     |P(p / q)| >= 1 / q^n,
//
// and combining the two bounds gives |alpha - p / q| >= 1 / (M * q^n) for
// every p / q in [alpha - 1, alpha + 1].  The finitely many p / q outside
// that interval are handled by shrinking c, yielding the strict inequality
// with c = 1 / (2 * M), say.
//
// The library does not yet formalize algebraic numbers of arbitrary degree
// (continued_fraction_periodic.ac has the quadratic case,
// `is_quadratic_irrational`, but no general "root of a degree-n integer
// polynomial" predicate on `Real`), nor the integer-value lemma
// `q^n * P(p / q) in Z` for a rational p / q.  A faithful formal statement
// needs the predicate "alpha is algebraic of degree n" — a root of a
// degree-n polynomial with integer coefficients, irreducible over the
// rationals — which the library does not yet provide, so the theorem is
// recorded here in prose for future work.

// ============================================================================
// Section 5: the irrationality of e
// ============================================================================

// The irrationality of e.  The library defines Euler's number
// `Real.e = exp(Real.1)` as the limit of the partial sums of the exponential
// series (real/exp.ac), and proves e lies strictly between two and three,
// but does not prove it irrational.  The statement below is recorded in
// approximation_deep.ac (Section 4) and is restated here.
//
// Fourier's classical proof runs as follows.  From the series
// e = sum_{k >= 0} 1 / k!, multiply by N!:
//
//     N! * e = sum_{k = 0}^{N} N! / k! + sum_{k > N} N! / k!.
//
// The first sum is an integer (N! / k! = (k + 1) ... N is a natural), while
// the tail satisfies 0 < sum_{k > N} N! / k! < 1: each term equals
// 1 / ((N + 1) ... k) <= 1 / (N + 1)^(k - N), and the geometric series
// bounds the tail by 1 / N.  Hence N! * e is strictly between the integer
// sum_{k = 0}^{N} N! / k! and that integer plus one, so it is not an
// integer.  Since N was arbitrary, and a rational a / b would make N! * e an
// integer for every N >= b (because b divides N!), e cannot be rational.
//
// The library lacks three ingredients for this proof, exactly as noted in
// approximation_deep.ac: the divisibility N! / k! for k <= N (a
// factorial-divisibility lemma), the geometric tail bound on the exp series
// in this form, and a "strictly between consecutive integers is not an
// integer" lemma for reals.  The statement is recorded here for future work.
//
// theorem approximation_theorems_e_irrational {
//     not exists(a: Nat, b: Nat) {
//         Nat.1 <= b and Real.e = Real.from_rat(Rat.from_nat(a) / Rat.from_nat(b))
//     }
// }

// ============================================================================
// Section 6: equidistribution of the fractional parts of n * alpha
// ============================================================================

// The equidistribution statement: for irrational alpha, the fractional parts
// of the multiples n * alpha are dense in the unit interval — for every
// x in [0, 1] and every positive tolerance there is a natural n whose
// fractional part {n * alpha} lies within the tolerance of x.
//
// Proof outline.  By Dirichlet's approximation theorem (Section 1), for
// every natural Q there is a q with 1 <= q <= Q and
// |q * alpha - p| < 1 / Q for some integer p; hence the distance from
// {q * alpha} to 0 is at most 1 / Q.  Choose such a q with the step
// delta = {q * alpha} (or its reflection 1 - delta, whichever is closer to
// zero) strictly positive and smaller than the given tolerance; alpha
// irrational keeps delta nonzero.  The points 0, delta, 2 * delta, ...,
// k * delta walk across [0, 1] in steps of size at most the tolerance
// (k * delta <= 1 < (k + 1) * delta), so some multiple k * q has fractional
// part {k * q * alpha} within the tolerance of x.  This is the classical
// "the multiples of an irrational are dense modulo one" argument; the full
// equidistribution theorem (the averages of f({n * alpha}) converge to the
// integral of f) is Weyl's theorem and is deeper.
//
// The library does not yet provide a floor or fractional-part operation on
// `Real` (there is no floor in real/), nor a density predicate for subsets
// of the unit interval phrased through fractional parts, so the statement is
// recorded here for future work.
//
// theorem approximation_theorems_fractional_parts_dense(alpha: Real) {
//     not exists(a: Nat, b: Nat) {
//         Nat.1 <= b and alpha = Real.from_rat(Rat.from_nat(a) / Rat.from_nat(b))
//     } implies forall(x: Real, eps: Real) {
//         Real.0 <= x and x <= Real.1 and Real.0 < eps implies
//         exists(n: Nat) {
//             exists(m: Nat) {
//                 from_nat[Real](n) * alpha - from_nat[Real](m) >= Real.0 and
//                 from_nat[Real](n) * alpha - from_nat[Real](m) <= Real.1 and
//                 (from_nat[Real](n) * alpha - from_nat[Real](m) - x).abs < eps
//             }
//         }
//     }
// }
