/// Number theory applications: concrete modular computations and classical
/// competition-style results.
///
/// This file collects the standard application results of the number theory
/// base: the concrete Fermat-computation examples `2^100 ≡ 2 (mod 7)` and
/// `3^1000 ≡ 1 (mod 11)`, the last digit of the tower `7^(7^7)` through the
/// period-4 pattern of powers of 7 modulo 10, the Chinese remainder theorem
/// example `x ≡ 2 (mod 3)`, `x ≡ 3 (mod 5)` with its unique solution `8`
/// modulo 15 (restated from crt_applications.ac), and the classical
/// competition fact that `x² ≡ 1 (mod p)` has exactly the two solutions
/// `x ≡ ±1 (mod p)` for an odd prime `p`.
///
/// The heavy lifting lives in `fermat.ac` (Fermat's little theorem and
/// Euclid's lemma), `totient.ac` (Euler's theorem), `modular_power.ac`
/// (exponent reduction and the period of 7 modulo 10), and
/// `crt_applications.ac` (the CRT example); this file instantiates and
/// restates those results on the concrete examples.

from nat import Nat
from nat import exp_add, exp_mul, exp_one, exp_zero, one_exp, pow_pow, sq_eq_mul,
    small_mod, mod_of_decomp, mod_lt, mod_of_zero, add_sub, suc_sub_one,
    sub_one_lt, sub_pos, div_mod_decomp, add_imp_sub_left, add_imp_sub, mul_comm,
    mul_one_left, mul_one_right, mul_two_left, distrib_left, distrib_right,
    mul_assoc, add_assoc, add_comm, add_zero_left, add_suc_right, add_suc_left,
    add_one_right, add_zero_right, one_plus_one, zero_or_suc, pos_of_ne_zero,
    zero_exp, div_imp_mod, div_sub_mod, lt_trans, lt_add_suc, lt_suc,
    lt_imp_lt_suc, lt_suc_right, lt_imp_lte_suc, lte_trans, lte_imp_not_lt,
    lt_not_ref, gcd_of_prime, divides_mul, divides_self, divides_lte,
    lte_mul_both, nat_mul_3_3, nat_mul_2_5, nat_mul_7_7, nat_mul_9_7,
    nat_mul_9_9, nat_mul_1_4, nat_mul_6_6, nat_mul_6_2, nat_mul_1_5,
    read_add_single, read_add_read, read_mul_single, read_read_carry
from number_theory.congruence import congr_mod_refl, congr_mod_symm,
    congr_mod_trans, congr_mod_mul, congr_mod_pow, congr_mod_add,
    congr_mod_zero_of_divides, divides_of_congr_mod_zero, mod_congr_mod_self
from number_theory.fermat import fermat_euler, prime_divides_mul,
    prime_does_not_divide_below
from number_theory.totient import coprime_below_prime
from number_theory.zsigmondy import seven_is_prime, five_is_prime, two_pow_four,
    three_pow_two, lt_ne, not_divides_of_lt
from number_theory.carmichael import two_is_prime, three_is_prime
from number_theory.carmichael_properties import eleven_is_prime
from number_theory.modular_power import exponent_reduction_fermat,
    seven_pow_period_residue, seven_pow_three_congr_three,
    seven_congr_three_mod_four, three_lt_ten
from number_theory.crt_applications import crt_example_solution_exists,
    crt_example_unique_mod_fifteen
numerals Nat

// ---------------------------------------------------------------------------
// Small arithmetic facts used by the concrete computations.
// ---------------------------------------------------------------------------

/// `6 + 4 = 10`.
theorem app_add_6_4 {
    Nat.6 + Nat.4 = Nat.10
} by {
    add_suc_right(Nat.6, Nat.3)
    Nat.6 + Nat.4 = (Nat.6 + Nat.3).suc
    add_suc_right(Nat.6, Nat.2)
    Nat.6 + Nat.3 = (Nat.6 + Nat.2).suc
    add_suc_right(Nat.6, Nat.1)
    Nat.6 + Nat.2 = (Nat.6 + Nat.1).suc
    add_one_right(Nat.6)
    Nat.6 + Nat.1 = Nat.7
    (Nat.6 + Nat.1).suc = Nat.7.suc
    Nat.7.suc = Nat.8
    Nat.6 + Nat.2 = Nat.8
    (Nat.6 + Nat.2).suc = Nat.8.suc
    Nat.8.suc = Nat.9
    Nat.6 + Nat.3 = Nat.9
    (Nat.6 + Nat.3).suc = Nat.9.suc
    Nat.9.suc = Nat.10
    Nat.6 + Nat.4 = Nat.10
}

/// `6 + 3 = 9`.
theorem app_add_6_3 {
    Nat.6 + Nat.3 = Nat.9
} by {
    add_suc_right(Nat.6, Nat.2)
    Nat.6 + Nat.3 = (Nat.6 + Nat.2).suc
    add_suc_right(Nat.6, Nat.1)
    Nat.6 + Nat.2 = (Nat.6 + Nat.1).suc
    add_one_right(Nat.6)
    Nat.6 + Nat.1 = Nat.7
    (Nat.6 + Nat.1).suc = Nat.7.suc
    Nat.7.suc = Nat.8
    Nat.6 + Nat.2 = Nat.8
    (Nat.6 + Nat.2).suc = Nat.8.suc
    Nat.8.suc = Nat.9
    Nat.6 + Nat.3 = Nat.9
}

/// `90 + 6 = 96`, through the decimal read: `9.read(0) + 6 = 9.read(6)`.
theorem app_add_90_6 {
    Nat.90 + Nat.6 = Nat.96
} by {
    Nat.9.read(Nat.0) = Nat.90
    read_add_single(Nat.9, Nat.0, Nat.6)
    Nat.9.read(Nat.0) + Nat.6 = Nat.9.read(Nat.0 + Nat.6)
    add_zero_left(Nat.6)
    Nat.0 + Nat.6 = Nat.6
    Nat.9.read(Nat.6) = Nat.96
    Nat.90 + Nat.6 = Nat.96
}

/// `90 + 10 = 100`, through the decimal read:
/// `9.read(0) + 1.read(0) = 10.read(0)`.
theorem app_add_90_10 {
    Nat.90 + Nat.10 = Nat.100
} by {
    Nat.9.read(Nat.0) = Nat.90
    Nat.1.read(Nat.0) = Nat.10
    read_add_read(Nat.9, Nat.0, Nat.1, Nat.0)
    Nat.9.read(Nat.0) + Nat.1.read(Nat.0) = (Nat.9 + Nat.1).read(Nat.0 + Nat.0)
    add_one_right(Nat.9)
    Nat.9 + Nat.1 = Nat.10
    add_zero_right(Nat.0)
    Nat.0 + Nat.0 = Nat.0
    Nat.10.read(Nat.0) = Nat.100
    Nat.90 + Nat.10 = Nat.100
}

/// `96 + 4 = 100`, by splitting `96 = 90 + 6`.
theorem app_add_96_four {
    Nat.96 + Nat.4 = Nat.100
} by {
    app_add_90_6
    Nat.96 = Nat.90 + Nat.6
    add_assoc(Nat.90, Nat.6, Nat.4)
    Nat.90 + Nat.6 + Nat.4 = Nat.90 + (Nat.6 + Nat.4)
    app_add_6_4
    Nat.6 + Nat.4 = Nat.10
    Nat.90 + (Nat.6 + Nat.4) = Nat.90 + Nat.10
    app_add_90_10
    Nat.90 + Nat.10 = Nat.100
    Nat.96 + Nat.4 = Nat.100
}

/// `60 + 36 = 96`, through the decimal read:
/// `6.read(0) + 3.read(6) = 9.read(6)`.
theorem app_add_60_36 {
    Nat.60 + Nat.36 = Nat.96
} by {
    Nat.6.read(Nat.0) = Nat.60
    Nat.3.read(Nat.6) = Nat.36
    read_add_read(Nat.6, Nat.0, Nat.3, Nat.6)
    Nat.6.read(Nat.0) + Nat.3.read(Nat.6) = (Nat.6 + Nat.3).read(Nat.0 + Nat.6)
    app_add_6_3
    Nat.6 + Nat.3 = Nat.9
    add_zero_left(Nat.6)
    Nat.0 + Nat.6 = Nat.6
    Nat.9.read(Nat.6) = Nat.96
    Nat.60 + Nat.36 = Nat.96
}

/// `6 · 10 = 60`: `6 · 10 = 12 · 5 = 1.read(2) · 5 = 5.read(10) = 6.read(0)`.
theorem app_mul_6_10 {
    Nat.6 * Nat.10 = Nat.60
} by {
    mul_assoc(Nat.6, Nat.2, Nat.5)
    Nat.6 * Nat.2 * Nat.5 = Nat.6 * (Nat.2 * Nat.5)
    nat_mul_6_2
    Nat.6 * Nat.2 = Nat.12
    Nat.12 * Nat.5 = Nat.6 * (Nat.2 * Nat.5)
    nat_mul_2_5
    Nat.2 * Nat.5 = Nat.10
    Nat.6 * (Nat.2 * Nat.5) = Nat.6 * Nat.10
    Nat.12 * Nat.5 = Nat.6 * Nat.10
    Nat.1.read(Nat.2) = Nat.12
    read_mul_single(Nat.1, Nat.2, Nat.5)
    Nat.1.read(Nat.2) * Nat.5 = (Nat.1 * Nat.5).read(Nat.2 * Nat.5)
    Nat.12 * Nat.5 = (Nat.1 * Nat.5).read(Nat.2 * Nat.5)
    nat_mul_1_5
    Nat.1 * Nat.5 = Nat.5
    nat_mul_2_5
    Nat.2 * Nat.5 = Nat.10
    Nat.12 * Nat.5 = Nat.5.read(Nat.10)
    Nat.10 * Nat.1 + Nat.0 = Nat.10
    read_read_carry(Nat.5, Nat.1, Nat.0)
    Nat.5.read(Nat.10 * Nat.1 + Nat.0) = (Nat.5 + Nat.1).read(Nat.0)
    Nat.5.read(Nat.10) = (Nat.5 + Nat.1).read(Nat.0)
    add_one_right(Nat.5)
    Nat.5 + Nat.1 = Nat.6
    Nat.5.read(Nat.10) = Nat.6.read(Nat.0)
    Nat.6.read(Nat.0) = Nat.60
    Nat.12 * Nat.5 = Nat.60
    Nat.6 * Nat.10 = Nat.60
}

/// `16 · 6 = 96`: `16 = 10 + 6`, so
/// `16 · 6 = 6 · 16 = 6 · 10 + 6 · 6 = 60 + 36 = 96`.
theorem app_sixteen_mul_six {
    Nat.16 * Nat.6 = Nat.96
} by {
    mul_comm(Nat.16, Nat.6)
    Nat.16 * Nat.6 = Nat.6 * Nat.16
    distrib_left(Nat.6, Nat.10, Nat.6)
    Nat.6 * (Nat.10 + Nat.6) = Nat.6 * Nat.10 + Nat.6 * Nat.6
    Nat.10 + Nat.6 = Nat.16
    Nat.6 * Nat.16 = Nat.6 * Nat.10 + Nat.6 * Nat.6
    app_mul_6_10
    Nat.6 * Nat.10 = Nat.60
    nat_mul_6_6
    Nat.6 * Nat.6 = Nat.36
    Nat.6 * Nat.16 = Nat.60 + Nat.36
    app_add_60_36
    Nat.60 + Nat.36 = Nat.96
    Nat.16 * Nat.6 = Nat.96
}

/// `100 = 16 · 6 + 4`: the decomposition of 100 used to reduce the exponent
/// of `2^100` modulo `6 = 7 - 1`.
theorem app_hundred_decomp {
    Nat.16 * Nat.6 + Nat.4 = Nat.100
} by {
    app_sixteen_mul_six
    Nat.16 * Nat.6 = Nat.96
    app_add_96_four
    Nat.96 + Nat.4 = Nat.100
    Nat.16 * Nat.6 + Nat.4 = Nat.100
}

/// `2 < 7`.
theorem app_two_lt_seven {
    Nat.2 < Nat.7
} by {
    lt_add_suc(Nat.2, Nat.4)
    Nat.2 < Nat.2 + Nat.5
    Nat.2 + Nat.5 = Nat.7
    Nat.2 < Nat.7
}

/// `3 < 11`.
theorem app_three_lt_eleven {
    Nat.3 < Nat.11
} by {
    lt_suc(Nat.3)
    Nat.3 < Nat.4
    lt_imp_lt_suc(Nat.3, Nat.4)
    Nat.3 < Nat.5
    lt_imp_lt_suc(Nat.3, Nat.5)
    Nat.3 < Nat.6
    lt_imp_lt_suc(Nat.3, Nat.6)
    Nat.3 < Nat.7
    lt_imp_lt_suc(Nat.3, Nat.7)
    Nat.3 < Nat.8
    lt_imp_lt_suc(Nat.3, Nat.8)
    Nat.3 < Nat.9
    lt_imp_lt_suc(Nat.3, Nat.9)
    Nat.3 < Nat.10
    lt_imp_lt_suc(Nat.3, Nat.10)
    Nat.3 < Nat.11
}

// ---------------------------------------------------------------------------
// Fermat's little theorem at the concrete primes.
// ---------------------------------------------------------------------------

/// `2^6 ≡ 1 (mod 7)`: Fermat's little theorem at the prime `7` with base `2`,
/// using `6 = 7 - 1`.
theorem two_pow_six_congr_one_mod_seven {
    Nat.2.pow(Nat.6).congr_mod(Nat.1, Nat.7)
} by {
    seven_is_prime
    Nat.7.is_prime
    Nat.1 <= Nat.2
    app_two_lt_seven
    Nat.2 < Nat.7
    coprime_below_prime(Nat.7, Nat.2)
    Nat.2.coprime(Nat.7)
    fermat_euler(Nat.7, Nat.2)
    Nat.2.pow(Nat.7 - Nat.1).congr_mod(Nat.1, Nat.7)
    suc_sub_one(Nat.6)
    Nat.7 - Nat.1 = Nat.6
    Nat.2.pow(Nat.6).congr_mod(Nat.1, Nat.7)
}

/// `2^100 ≡ 2 (mod 7)`: since `100 = 4 + 16·6`, Fermat's `2^6 ≡ 1` reduces
/// the exponent modulo 6, and `2^4 = 16 ≡ 2 (mod 7)`.
theorem two_pow_hundred_congr_two_mod_seven {
    Nat.2.pow(Nat.100).congr_mod(Nat.2, Nat.7)
} by {
    seven_is_prime
    Nat.7.is_prime
    Nat.1 <= Nat.2
    app_two_lt_seven
    Nat.2 < Nat.7
    coprime_below_prime(Nat.7, Nat.2)
    Nat.2.coprime(Nat.7)
    suc_sub_one(Nat.6)
    Nat.7 - Nat.1 = Nat.6
    app_hundred_decomp
    Nat.16 * Nat.6 + Nat.4 = Nat.100
    add_comm(Nat.16 * Nat.6, Nat.4)
    Nat.4 + Nat.16 * Nat.6 = Nat.100
    Nat.4 + Nat.16 * (Nat.7 - Nat.1) = Nat.100
    Nat.100 = Nat.4 + Nat.16 * (Nat.7 - Nat.1)
    exponent_reduction_fermat(Nat.7, Nat.2, Nat.100, Nat.4, Nat.16)
    Nat.2.pow(Nat.100).congr_mod(Nat.2.pow(Nat.4), Nat.7)
    two_pow_four
    Nat.2.pow(Nat.4) = Nat.16
    Nat.2 * Nat.7 + Nat.2 = Nat.16
    app_two_lt_seven
    Nat.2 < Nat.7
    mod_of_decomp(Nat.2, Nat.2, Nat.7)
    (Nat.2 * Nat.7 + Nat.2).mod(Nat.7) = Nat.2
    Nat.16.mod(Nat.7) = Nat.2
    small_mod(Nat.2, Nat.7)
    Nat.2.mod(Nat.7) = Nat.2
    Nat.16.mod(Nat.7) = Nat.2.mod(Nat.7)
    Nat.2.pow(Nat.4).congr_mod(Nat.2, Nat.7)
    congr_mod_trans(Nat.2.pow(Nat.100), Nat.2.pow(Nat.4), Nat.2, Nat.7)
    Nat.2.pow(Nat.100).congr_mod(Nat.2, Nat.7)
}

/// The concrete computation: the remainder of `2^100` modulo `7` is `2`.
theorem two_pow_hundred_mod_seven {
    Nat.2.pow(Nat.100).mod(Nat.7) = Nat.2
} by {
    two_pow_hundred_congr_two_mod_seven
    Nat.2.pow(Nat.100).congr_mod(Nat.2, Nat.7)
    app_two_lt_seven
    Nat.2 < Nat.7
    small_mod(Nat.2, Nat.7)
    Nat.2.mod(Nat.7) = Nat.2
    Nat.2.pow(Nat.100).mod(Nat.7) = Nat.2
}

/// `3^10 ≡ 1 (mod 11)`: Fermat's little theorem at the prime `11` with base
/// `3`, using `10 = 11 - 1`.
theorem three_pow_ten_congr_one_mod_eleven {
    Nat.3.pow(Nat.10).congr_mod(Nat.1, Nat.11)
} by {
    eleven_is_prime
    Nat.11.is_prime
    Nat.1 <= Nat.3
    app_three_lt_eleven
    Nat.3 < Nat.11
    coprime_below_prime(Nat.11, Nat.3)
    Nat.3.coprime(Nat.11)
    fermat_euler(Nat.11, Nat.3)
    Nat.3.pow(Nat.11 - Nat.1).congr_mod(Nat.1, Nat.11)
    suc_sub_one(Nat.10)
    Nat.11 - Nat.1 = Nat.10
    Nat.3.pow(Nat.10).congr_mod(Nat.1, Nat.11)
}

/// `3^1000 ≡ 1 (mod 11)`: since `1000 = 0 + 100·10`, Fermat's `3^10 ≡ 1`
/// reduces the exponent modulo 10 to zero.
theorem three_pow_thousand_congr_one_mod_eleven {
    Nat.3.pow(Nat.1000).congr_mod(Nat.1, Nat.11)
} by {
    eleven_is_prime
    Nat.11.is_prime
    Nat.1 <= Nat.3
    app_three_lt_eleven
    Nat.3 < Nat.11
    coprime_below_prime(Nat.11, Nat.3)
    Nat.3.coprime(Nat.11)
    suc_sub_one(Nat.10)
    Nat.11 - Nat.1 = Nat.10
    Nat.100 * Nat.10 = Nat.1000
    add_zero_left(Nat.100 * Nat.10)
    Nat.0 + Nat.100 * Nat.10 = Nat.100 * Nat.10
    Nat.0 + Nat.100 * (Nat.11 - Nat.1) = Nat.1000
    Nat.1000 = Nat.0 + Nat.100 * (Nat.11 - Nat.1)
    exponent_reduction_fermat(Nat.11, Nat.3, Nat.1000, Nat.0, Nat.100)
    Nat.3.pow(Nat.1000).congr_mod(Nat.3.pow(Nat.0), Nat.11)
    exp_zero(Nat.3)
    Nat.3.pow(Nat.0) = Nat.1
    Nat.3.pow(Nat.1000).congr_mod(Nat.1, Nat.11)
}

/// The concrete computation: the remainder of `3^1000` modulo `11` is `1`.
theorem three_pow_thousand_mod_eleven {
    Nat.3.pow(Nat.1000).mod(Nat.11) = Nat.1
} by {
    three_pow_thousand_congr_one_mod_eleven
    Nat.3.pow(Nat.1000).congr_mod(Nat.1, Nat.11)
    Nat.1 < Nat.11
    small_mod(Nat.1, Nat.11)
    Nat.1.mod(Nat.11) = Nat.1
    Nat.3.pow(Nat.1000).mod(Nat.11) = Nat.1
}

// ---------------------------------------------------------------------------
// The last digit of 7^(7^7).
// ---------------------------------------------------------------------------

/// `7^7 ≡ 3 (mod 4)`: `7 ≡ 3 (mod 4)` and `3^2 ≡ 1 (mod 4)`, so
/// `3^7 = (3^2)^3 · 3 ≡ 3 (mod 4)`.
theorem seven_pow_seven_congr_three_mod_four {
    Nat.7.pow(Nat.7).congr_mod(Nat.3, Nat.4)
} by {
    seven_congr_three_mod_four
    Nat.7.congr_mod(Nat.3, Nat.4)
    congr_mod_pow(Nat.7, Nat.3, Nat.4, Nat.7)
    Nat.7.pow(Nat.7).congr_mod(Nat.3.pow(Nat.7), Nat.4)
    three_pow_two
    Nat.3.pow(Nat.2) = Nat.9
    Nat.2 * Nat.4 + Nat.1 = Nat.9
    Nat.1 < Nat.4
    mod_of_decomp(Nat.2, Nat.1, Nat.4)
    (Nat.2 * Nat.4 + Nat.1).mod(Nat.4) = Nat.1
    Nat.9.mod(Nat.4) = Nat.1
    small_mod(Nat.1, Nat.4)
    Nat.1.mod(Nat.4) = Nat.1
    Nat.9.mod(Nat.4) = Nat.1.mod(Nat.4)
    Nat.9.congr_mod(Nat.1, Nat.4)
    congr_mod_pow(Nat.9, Nat.1, Nat.4, Nat.3)
    Nat.9.pow(Nat.3).congr_mod(Nat.1.pow(Nat.3), Nat.4)
    one_exp(Nat.3)
    Nat.1.pow(Nat.3) = Nat.1
    Nat.9.pow(Nat.3).congr_mod(Nat.1, Nat.4)
    exp_mul(Nat.3, Nat.2, Nat.3)
    Nat.3.pow(Nat.2 * Nat.3) = Nat.3.pow(Nat.2).pow(Nat.3)
    Nat.2 * Nat.3 = Nat.6
    Nat.3.pow(Nat.6) = Nat.3.pow(Nat.2).pow(Nat.3)
    Nat.3.pow(Nat.6) = Nat.9.pow(Nat.3)
    Nat.9.pow(Nat.3).congr_mod(Nat.1, Nat.4)
    Nat.3.pow(Nat.6).congr_mod(Nat.1, Nat.4)
    exp_add(Nat.3, Nat.6, Nat.1)
    Nat.3.pow(Nat.6 + Nat.1) = Nat.3.pow(Nat.6) * Nat.3.pow(Nat.1)
    Nat.6 + Nat.1 = Nat.7
    Nat.3.pow(Nat.7) = Nat.3.pow(Nat.6) * Nat.3.pow(Nat.1)
    exp_one(Nat.3)
    Nat.3.pow(Nat.1) = Nat.3
    Nat.3.pow(Nat.7) = Nat.3.pow(Nat.6) * Nat.3
    congr_mod_refl(Nat.3, Nat.4)
    congr_mod_mul(Nat.3.pow(Nat.6), Nat.3, Nat.1, Nat.3, Nat.4)
    (Nat.3.pow(Nat.6) * Nat.3).congr_mod(Nat.1 * Nat.3, Nat.4)
    Nat.1 * Nat.3 = Nat.3
    Nat.3.pow(Nat.7).congr_mod(Nat.3, Nat.4)
    congr_mod_trans(Nat.7.pow(Nat.7), Nat.3.pow(Nat.7), Nat.3, Nat.4)
    Nat.7.pow(Nat.7).congr_mod(Nat.3, Nat.4)
}

/// The remainder of the exponent `7^7` modulo `4` is `3`.
theorem seven_pow_seven_mod_four_three {
    Nat.7.pow(Nat.7).mod(Nat.4) = Nat.3
} by {
    seven_pow_seven_congr_three_mod_four
    Nat.7.pow(Nat.7).congr_mod(Nat.3, Nat.4)
    Nat.3 < Nat.4
    small_mod(Nat.3, Nat.4)
    Nat.3.mod(Nat.4) = Nat.3
    Nat.7.pow(Nat.7).mod(Nat.4) = Nat.3
}

/// `7^(7^7) ≡ 3 (mod 10)`: the exponent `7^7` is three more than a multiple
/// of four, so the period-4 pattern of powers of 7 modulo 10 gives the same
/// residue as `7^3`, and `7^3 ≡ 3 (mod 10)`.
theorem seven_pow_seven_pow_seven_congr_three_mod_ten {
    Nat.7.pow(Nat.7.pow(Nat.7)).congr_mod(Nat.3, Nat.10)
} by {
    seven_pow_seven_mod_four_three
    Nat.7.pow(Nat.7).mod(Nat.4) = Nat.3
    div_mod_decomp(Nat.7.pow(Nat.7), Nat.4)
    Nat.7.pow(Nat.7).div(Nat.4) * Nat.4 + Nat.7.pow(Nat.7).mod(Nat.4) =
        Nat.7.pow(Nat.7)
    Nat.7.pow(Nat.7).div(Nat.4) * Nat.4 + Nat.3 = Nat.7.pow(Nat.7)
    mul_comm(Nat.7.pow(Nat.7).div(Nat.4), Nat.4)
    Nat.4 * Nat.7.pow(Nat.7).div(Nat.4) + Nat.3 = Nat.7.pow(Nat.7)
    seven_pow_period_residue(Nat.7.pow(Nat.7).div(Nat.4), Nat.3)
    Nat.7.pow(Nat.4 * Nat.7.pow(Nat.7).div(Nat.4) + Nat.3).congr_mod(Nat.7.pow(Nat.3), Nat.10)
    Nat.7.pow(Nat.7.pow(Nat.7)).congr_mod(Nat.7.pow(Nat.3), Nat.10)
    seven_pow_three_congr_three
    Nat.7.pow(Nat.3).congr_mod(Nat.3, Nat.10)
    congr_mod_trans(Nat.7.pow(Nat.7.pow(Nat.7)), Nat.7.pow(Nat.3), Nat.3, Nat.10)
    Nat.7.pow(Nat.7.pow(Nat.7)).congr_mod(Nat.3, Nat.10)
}

/// The last digit of the tower `7^7^7` is `3`.
theorem seven_pow_seven_pow_seven_last_digit {
    Nat.7.pow(Nat.7.pow(Nat.7)).mod(Nat.10) = Nat.3
} by {
    seven_pow_seven_pow_seven_congr_three_mod_ten
    Nat.7.pow(Nat.7.pow(Nat.7)).congr_mod(Nat.3, Nat.10)
    three_lt_ten
    Nat.3 < Nat.10
    small_mod(Nat.3, Nat.10)
    Nat.3.mod(Nat.10) = Nat.3
    Nat.7.pow(Nat.7.pow(Nat.7)).mod(Nat.10) = Nat.3
}

// ---------------------------------------------------------------------------
// The Chinese remainder theorem example, restated.
// ---------------------------------------------------------------------------

/// The classic CRT example, restated: the system `x ≡ 2 (mod 3)`,
/// `x ≡ 3 (mod 5)` has a simultaneous solution.
theorem crt_example_solution_exists_app {
    exists(x: Nat) { x.congr_mod(Nat.2, Nat.3) and x.congr_mod(Nat.3, Nat.5) }
} by {
    crt_example_solution_exists
}

/// The classic CRT example, restated: every solution of the system
/// `x ≡ 2 (mod 3)`, `x ≡ 3 (mod 5)` is congruent to `8` modulo `15`.
theorem crt_example_unique_mod_fifteen_app(c: Nat) {
    c.congr_mod(Nat.2, Nat.3) and c.congr_mod(Nat.3, Nat.5)
        implies c.congr_mod(Nat.8, Nat.15)
} by {
    if c.congr_mod(Nat.2, Nat.3) and c.congr_mod(Nat.3, Nat.5) {
        crt_example_unique_mod_fifteen(c)
        c.congr_mod(Nat.2, Nat.3) and c.congr_mod(Nat.3, Nat.5) implies c.congr_mod(Nat.8, Nat.15)
        c.congr_mod(Nat.8, Nat.15)
    }
}

// ---------------------------------------------------------------------------
// The solutions of x² ≡ 1 (mod p) for an odd prime p.
// ---------------------------------------------------------------------------

/// `(k + 1)² - 1 = k · (k + 2)`: the difference of squares for consecutive
/// integers.  Since `(k + 1)² = k² + 2k + 1`, subtracting 1 leaves
/// `k² + 2k = k(k + 2)`.
theorem sq_suc_sub_one_factor(k: Nat) {
    (k + Nat.1).pow(Nat.2) - Nat.1 = k * (k + Nat.2)
} by {
    sq_eq_mul(k + Nat.1)
    (k + Nat.1).pow(Nat.2) = (k + Nat.1) * (k + Nat.1)
    distrib_left(k + Nat.1, k, Nat.1)
    (k + Nat.1) * (k + Nat.1) = (k + Nat.1) * k + (k + Nat.1) * Nat.1
    mul_one_right(k + Nat.1)
    (k + Nat.1) * Nat.1 = k + Nat.1
    distrib_right(k, Nat.1, k)
    (k + Nat.1) * k = k * k + Nat.1 * k
    mul_one_left(k)
    Nat.1 * k = k
    sq_eq_mul(k)
    k.pow(Nat.2) = k * k
    k * k = k.pow(Nat.2)
    (k + Nat.1).pow(Nat.2) = k.pow(Nat.2) + k + (k + Nat.1)
    (k + Nat.1).pow(Nat.2) = k.pow(Nat.2) + Nat.2 * k + Nat.1
    add_imp_sub(k.pow(Nat.2) + Nat.2 * k, Nat.1, (k + Nat.1).pow(Nat.2))
    (k + Nat.1).pow(Nat.2) - Nat.1 = k.pow(Nat.2) + Nat.2 * k
    distrib_left(k, k, Nat.2)
    k * (k + Nat.2) = k * k + k * Nat.2
    mul_comm(k, Nat.2)
    k * Nat.2 = Nat.2 * k
    k * (k + Nat.2) = k * k + Nat.2 * k
    k * (k + Nat.2) = k.pow(Nat.2) + Nat.2 * k
    (k + Nat.1).pow(Nat.2) - Nat.1 = k * (k + Nat.2)
}

/// `(p - 1)² - 1 = (p - 2) · p` for `p ≥ 2`: the difference of squares at
/// `k = p - 2`.
theorem pred_sq_sub_one(p: Nat) {
    Nat.2 <= p implies (p - Nat.1).pow(Nat.2) - Nat.1 = (p - Nat.2) * p
} by {
    if Nat.2 <= p {
        add_sub(p, Nat.2)
        p - Nat.2 + Nat.2 = p
        sq_suc_sub_one_factor(p - Nat.2)
        ((p - Nat.2) + Nat.1).pow(Nat.2) - Nat.1 = (p - Nat.2) * ((p - Nat.2) + Nat.2)
        add_assoc(p - Nat.2, Nat.1, Nat.1)
        (p - Nat.2) + Nat.1 + Nat.1 = (p - Nat.2) + (Nat.1 + Nat.1)
        one_plus_one
        Nat.1 + Nat.1 = Nat.2
        (p - Nat.2) + (Nat.1 + Nat.1) = (p - Nat.2) + Nat.2
        (p - Nat.2) + Nat.2 = p
        (p - Nat.2) + Nat.1 + Nat.1 = p
        add_imp_sub((p - Nat.2) + Nat.1, Nat.1, p)
        p - Nat.1 = (p - Nat.2) + Nat.1
        (p - Nat.2) + Nat.1 = p - Nat.1
        (p - Nat.2) + Nat.2 = p
        (p - Nat.2) * ((p - Nat.2) + Nat.2) = (p - Nat.2) * p
        (p - Nat.1).pow(Nat.2) - Nat.1 = (p - Nat.2) * p
    }
}

/// `(p - 1)² ≡ 1 (mod p)` for an odd prime `p`: expanding `(p - 1)²` as
/// `p·(p - 2) + 1` leaves remainder 1.
theorem pred_square_congr_one(p: Nat) {
    p.is_prime and Nat.2 < p implies (p - Nat.1).pow(Nat.2).congr_mod(Nat.1, p)
} by {
    if p.is_prime and Nat.2 < p {
        Nat.2 <= p
        pred_sq_sub_one(p)
        (p - Nat.1).pow(Nat.2) - Nat.1 = (p - Nat.2) * p
        mul_comm(p - Nat.2, p)
        (p - Nat.2) * p = p * (p - Nat.2)
        (p - Nat.1).pow(Nat.2) - Nat.1 = p * (p - Nat.2)
        Nat.1 < p
        sub_pos(p, Nat.1)
        Nat.0 < p - Nat.1
        lt_imp_lte_suc(Nat.0, p - Nat.1)
        Nat.1 <= p - Nat.1
        lte_mul_both(p - Nat.1, Nat.1, p - Nat.1)
        (p - Nat.1) * Nat.1 <= (p - Nat.1) * (p - Nat.1)
        mul_one_right(p - Nat.1)
        (p - Nat.1) * Nat.1 = p - Nat.1
        sq_eq_mul(p - Nat.1)
        (p - Nat.1).pow(Nat.2) = (p - Nat.1) * (p - Nat.1)
        p - Nat.1 <= (p - Nat.1).pow(Nat.2)
        lte_trans(Nat.1, p - Nat.1, (p - Nat.1).pow(Nat.2))
        Nat.1 <= (p - Nat.1).pow(Nat.2)
        add_sub((p - Nat.1).pow(Nat.2), Nat.1)
        (p - Nat.1).pow(Nat.2) - Nat.1 + Nat.1 = (p - Nat.1).pow(Nat.2)
        p * (p - Nat.2) + Nat.1 = (p - Nat.1).pow(Nat.2)
        mod_of_decomp(p - Nat.2, Nat.1, p)
        (p * (p - Nat.2) + Nat.1).mod(p) = Nat.1.mod(p)
        (p - Nat.1).pow(Nat.2).mod(p) = Nat.1.mod(p)
        (p - Nat.1).pow(Nat.2).congr_mod(Nat.1, p)
    }
}

/// A divisor never divides a smaller natural strictly below it, unless the
/// smaller natural is zero: `d | m` and `m < d` force `m = 0`.
theorem divides_below_imp_zero(d: Nat, m: Nat) {
    d.divides(m) and m < d implies m = Nat.0
} by {
    if d.divides(m) and m < d {
        if m = Nat.0 {
        } else {
            m != Nat.0
            pos_of_ne_zero(m)
            Nat.0 < m
            not_divides_of_lt(d, m)
            not d.divides(m)
            false
        }
        m = Nat.0
    }
}

/// A residue `r < p` with `r² ≡ 1 (mod p)` for an odd prime `p` is either
/// `1` or `p - 1`.  Since `p | (r² - 1) = (r - 1)(r + 1)` and `p` is prime,
/// one of the factors vanishes modulo `p`; the factor `r + 1` is below `p`
/// (or equal to it), which pins `r` down to `p - 1` or `1`.
theorem sq_congr_one_mod_prime_residue(p: Nat, r: Nat) {
    p.is_prime and Nat.2 < p and r < p and r.pow(Nat.2).congr_mod(Nat.1, p)
        implies r = Nat.1 or r = p - Nat.1
} by {
    if p.is_prime and Nat.2 < p and r < p and r.pow(Nat.2).congr_mod(Nat.1, p) {
        Nat.1 < p
        small_mod(Nat.1, p)
        Nat.1.mod(p) = Nat.1
        r.pow(Nat.2).congr_mod(Nat.1, p)
        r.pow(Nat.2).mod(p) = Nat.1.mod(p)
        r.pow(Nat.2).mod(p) = Nat.1
        if r = Nat.0 {
            sq_eq_mul(Nat.0)
            Nat.0.pow(Nat.2) = Nat.0 * Nat.0
            Nat.0 * Nat.0 = Nat.0
            Nat.0.pow(Nat.2) = Nat.0
            Nat.0.mod(p) = Nat.1
            mod_of_zero(p)
            Nat.0.mod(p) = Nat.0
            Nat.0 = Nat.1
            false
        }
        r != Nat.0
        zero_or_suc(r)
        r = Nat.0 or exists(k: Nat) { r = k.suc }
        let k: Nat satisfy { r = k.suc }
        r = k.suc
        add_suc_right(k, Nat.0)
        k + Nat.1 = (k + Nat.0).suc
        add_zero_right(k)
        k + Nat.0 = k
        (k + Nat.0).suc = k.suc
        k + Nat.1 = k.suc
        r = k + Nat.1
        suc_sub_one(k)
        k.suc - Nat.1 = k
        r - Nat.1 = k
        add_suc_left(k, Nat.1)
        k.suc + Nat.1 = (k + Nat.1).suc
        (k + Nat.1).suc = k + Nat.1 + Nat.1
        k + Nat.1 + Nat.1 = k + Nat.2
        r + Nat.1 = k + Nat.2
        div_sub_mod(r.pow(Nat.2), p)
        p.divides(r.pow(Nat.2) - r.pow(Nat.2).mod(p))
        r.pow(Nat.2) - r.pow(Nat.2).mod(p) = r.pow(Nat.2) - Nat.1
        p.divides(r.pow(Nat.2) - Nat.1)
        sq_suc_sub_one_factor(k)
        (k + Nat.1).pow(Nat.2) - Nat.1 = k * (k + Nat.2)
        r.pow(Nat.2) - Nat.1 = k * (k + Nat.2)
        p.divides(k * (k + Nat.2))
        lt_imp_lte_suc(r, p)
        r.suc <= p
        add_one_right(r)
        r + Nat.1 = r.suc
        r + Nat.1 <= p
        if r + Nat.1 = p {
            add_imp_sub(r, Nat.1, p)
            p - Nat.1 = r
            r = p - Nat.1
            r = Nat.1 or r = p - Nat.1
        } else {
            r + Nat.1 < p
            Nat.0 < r + Nat.1
            prime_does_not_divide_below(p, r + Nat.1)
            not p.divides(r + Nat.1)
            prime_divides_mul(p, k, k + Nat.2)
            p.divides(k) or p.divides(k + Nat.2)
            if p.divides(k + Nat.2) {
                false
            }
            p.divides(k)
            pos_of_ne_zero(r)
            Nat.0 < r
            sub_one_lt(r)
            r - Nat.1 < r
            k < r
            lt_trans(k, r, p)
            k < p
            divides_below_imp_zero(p, k)
            k = Nat.0
            r = k + Nat.1
            Nat.0 + Nat.1 = Nat.1
            r = Nat.1
            r = Nat.1 or r = p - Nat.1
        }
    }
}

/// Forward direction: a solution of `x² ≡ 1 (mod p)` for an odd prime `p` is
/// congruent to `1` or to `p - 1` modulo `p`.  Reducing `x` to its remainder
/// `r = x mod p` below `p`, the classification of
/// `sq_congr_one_mod_prime_residue` applies.
theorem sq_congr_one_prime_solutions(p: Nat, x: Nat) {
    p.is_prime and Nat.2 < p and x.pow(Nat.2).congr_mod(Nat.1, p)
        implies (x.congr_mod(Nat.1, p) or x.congr_mod(p - Nat.1, p))
} by {
    if p.is_prime and Nat.2 < p and x.pow(Nat.2).congr_mod(Nat.1, p) {
        mod_congr_mod_self(x, p)
        x.mod(p).congr_mod(x, p)
        congr_mod_symm(x.mod(p), x, p)
        x.congr_mod(x.mod(p), p)
        congr_mod_pow(x, x.mod(p), p, Nat.2)
        x.pow(Nat.2).congr_mod(x.mod(p).pow(Nat.2), p)
        congr_mod_symm(x.pow(Nat.2), x.mod(p).pow(Nat.2), p)
        x.mod(p).pow(Nat.2).congr_mod(x.pow(Nat.2), p)
        congr_mod_trans(x.mod(p).pow(Nat.2), x.pow(Nat.2), Nat.1, p)
        x.mod(p).pow(Nat.2).congr_mod(Nat.1, p)
        p != Nat.0
        mod_lt(x, p)
        x.mod(p) < p
        sq_congr_one_mod_prime_residue(p, x.mod(p))
        x.mod(p) = Nat.1 or x.mod(p) = p - Nat.1
        if x.mod(p) = Nat.1 {
            congr_mod_trans(x, x.mod(p), Nat.1, p)
            x.congr_mod(Nat.1, p)
            x.congr_mod(Nat.1, p) or x.congr_mod(p - Nat.1, p)
        } else {
            x.mod(p) = p - Nat.1
            congr_mod_trans(x, x.mod(p), p - Nat.1, p)
            x.congr_mod(p - Nat.1, p)
            x.congr_mod(Nat.1, p) or x.congr_mod(p - Nat.1, p)
        }
    }
}

/// Backward direction: if `x` is congruent to `1` or to `p - 1` modulo an odd
/// prime `p`, then `x² ≡ 1 (mod p)`, since `1² = 1` and `(p - 1)² ≡ 1`.
theorem sq_congr_one_prime_roots(p: Nat, x: Nat) {
    p.is_prime and Nat.2 < p and (x.congr_mod(Nat.1, p) or x.congr_mod(p - Nat.1, p))
        implies x.pow(Nat.2).congr_mod(Nat.1, p)
} by {
    if p.is_prime and Nat.2 < p and (x.congr_mod(Nat.1, p) or x.congr_mod(p - Nat.1, p)) {
        if x.congr_mod(Nat.1, p) {
            congr_mod_pow(x, Nat.1, p, Nat.2)
            x.pow(Nat.2).congr_mod(Nat.1.pow(Nat.2), p)
            sq_eq_mul(Nat.1)
            Nat.1.pow(Nat.2) = Nat.1 * Nat.1
            Nat.1 * Nat.1 = Nat.1
            Nat.1.pow(Nat.2) = Nat.1
            x.pow(Nat.2).congr_mod(Nat.1, p)
        } else {
            congr_mod_pow(x, p - Nat.1, p, Nat.2)
            x.pow(Nat.2).congr_mod((p - Nat.1).pow(Nat.2), p)
            pred_square_congr_one(p)
            (p - Nat.1).pow(Nat.2).congr_mod(Nat.1, p)
            congr_mod_trans(x.pow(Nat.2), (p - Nat.1).pow(Nat.2), Nat.1, p)
            x.pow(Nat.2).congr_mod(Nat.1, p)
        }
    }
}

/// `x² ≡ 1 (mod p)` if and only if `x ≡ 1 (mod p)` or `x ≡ p - 1 (mod p)`,
/// for an odd prime `p`.  This combines the two directions
/// `sq_congr_one_prime_solutions` and `sq_congr_one_prime_roots`.
theorem sq_congr_one_prime_residues(p: Nat, x: Nat) {
    p.is_prime and Nat.2 < p implies
        (x.pow(Nat.2).congr_mod(Nat.1, p)
            = (x.congr_mod(Nat.1, p) or x.congr_mod(p - Nat.1, p)))
} by {
    if p.is_prime and Nat.2 < p {
        if x.pow(Nat.2).congr_mod(Nat.1, p) {
            sq_congr_one_prime_solutions(p, x)
            x.congr_mod(Nat.1, p) or x.congr_mod(p - Nat.1, p)
            x.pow(Nat.2).congr_mod(Nat.1, p) = (x.congr_mod(Nat.1, p) or x.congr_mod(p - Nat.1, p))
        }
        if not x.pow(Nat.2).congr_mod(Nat.1, p) {
            if x.congr_mod(Nat.1, p) or x.congr_mod(p - Nat.1, p) {
                sq_congr_one_prime_roots(p, x)
                x.pow(Nat.2).congr_mod(Nat.1, p)
                false
            }
            x.pow(Nat.2).congr_mod(Nat.1, p) = (x.congr_mod(Nat.1, p) or x.congr_mod(p - Nat.1, p))
        }
    }
}

/// Below the modulus `p`, the solutions of `x² ≡ 1 (mod p)` are exactly the
/// two representatives `1` and `p - 1`.
theorem sq_congr_one_prime_below(p: Nat, x: Nat) {
    p.is_prime and Nat.2 < p and x < p implies
        (x.pow(Nat.2).congr_mod(Nat.1, p) = (x = Nat.1 or x = p - Nat.1))
} by {
    if p.is_prime and Nat.2 < p and x < p {
        sq_congr_one_prime_residues(p, x)
        x.pow(Nat.2).congr_mod(Nat.1, p) = (x.congr_mod(Nat.1, p) or x.congr_mod(p - Nat.1, p))
        small_mod(x, p)
        x.mod(p) = x
        Nat.1 < p
        small_mod(Nat.1, p)
        Nat.1.mod(p) = Nat.1
        x.congr_mod(Nat.1, p) = (x = Nat.1)
        p != Nat.0
        sub_one_lt(p)
        p - Nat.1 < p
        small_mod(p - Nat.1, p)
        (p - Nat.1).mod(p) = p - Nat.1
        x.congr_mod(p - Nat.1, p) = (x = p - Nat.1)
        x.pow(Nat.2).congr_mod(Nat.1, p) = (x = Nat.1 or x = p - Nat.1)
    }
}

/// For an odd prime `p`, the two representatives `1` and `p - 1` are
/// distinct: `p - 1 = 1` would force `p = 2`.
theorem sq_congr_one_prime_residues_distinct(p: Nat) {
    p.is_prime and Nat.2 < p implies Nat.1 != p - Nat.1
} by {
    if p.is_prime and Nat.2 < p {
        Nat.1 < p
        Nat.1 <= p
        if Nat.1 = p - Nat.1 {
            add_sub(p, Nat.1)
            p - Nat.1 + Nat.1 = p
            Nat.1 + Nat.1 = p
            one_plus_one
            Nat.1 + Nat.1 = Nat.2
            p = Nat.2
            Nat.2 < Nat.2
            lt_not_ref(Nat.2)
            false
        }
        Nat.1 != p - Nat.1
    }
}

/// An odd prime `p` has exactly two square roots of 1 modulo `p`: the
/// residue classes `1` and `p - 1` both square to 1, and every solution
/// below `p` is one of them.
theorem sq_congr_one_prime_two_solutions(p: Nat) {
    p.is_prime and Nat.2 < p implies
        exists(a: Nat, b: Nat) {
            a != b and a < p and b < p
            and a.pow(Nat.2).congr_mod(Nat.1, p)
            and b.pow(Nat.2).congr_mod(Nat.1, p)
            and forall(x: Nat) {
                x < p and x.pow(Nat.2).congr_mod(Nat.1, p) implies (x = a or x = b)
            }
        }
} by {
    if p.is_prime and Nat.2 < p {
        sq_congr_one_prime_residues_distinct(p)
        Nat.1 != p - Nat.1
        Nat.1 < p
        sq_eq_mul(Nat.1)
        Nat.1.pow(Nat.2) = Nat.1 * Nat.1
        Nat.1 * Nat.1 = Nat.1
        Nat.1.pow(Nat.2) = Nat.1
        congr_mod_refl(Nat.1, p)
        Nat.1.congr_mod(Nat.1, p)
        Nat.1.pow(Nat.2).congr_mod(Nat.1, p)
        p != Nat.0
        sub_one_lt(p)
        p - Nat.1 < p
        pred_square_congr_one(p)
        (p - Nat.1).pow(Nat.2).congr_mod(Nat.1, p)
        forall(x: Nat) {
            if x < p and x.pow(Nat.2).congr_mod(Nat.1, p) {
                sq_congr_one_prime_below(p, x)
                x.pow(Nat.2).congr_mod(Nat.1, p) = (x = Nat.1 or x = p - Nat.1)
                x = Nat.1 or x = p - Nat.1
                x = Nat.1 or x = p - Nat.1
            }
        }
        exists(a: Nat, b: Nat) {
            a != b and a < p and b < p
            and a.pow(Nat.2).congr_mod(Nat.1, p)
            and b.pow(Nat.2).congr_mod(Nat.1, p)
            and forall(x: Nat) {
                x < p and x.pow(Nat.2).congr_mod(Nat.1, p) implies (x = a or x = b)
            }
        }
    }
}
