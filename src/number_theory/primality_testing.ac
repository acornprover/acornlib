/// Primality testing.
///
/// This file collects the classical primality tests built on Fermat's little
/// theorem:
///
///   (a) the Fermat test: for a prime `p` and a base `a` coprime to `p`
///       (equivalently `gcd(a, p) = 1`), the power `a^(p - 1)` is congruent
///       to `1` modulo `p` (fermat.ac);
///   (b) the converse of the Fermat test for coprime bases is false: the
///       composite number 561 satisfies `a^560 ≡ 1 (mod 561)` for every unit
///       `a` modulo 561 — 561 is a Carmichael number (carmichael.ac,
///       carmichael_properties.ac);
///   (c) the base-2 Fermat test: for an odd prime `n`, the power
///       `2^(n - 1)` is congruent to `1` modulo `n`
///       (fermat_consequences.ac);
///   (d) the two-divisor test: `n` is prime exactly when it has exactly two
///       positive divisors, i.e. `τ(n) = 2` (tau_multiplicative.ac);
///   (e) trial division: `n` is prime exactly when no prime `p <= sqrt(n)`
///       divides `n` (stated here; the classical direction — a composite
///       number has a prime factor at most `sqrt(n)` — is proved).
///
/// The Fermat test and its base-2 form give necessary conditions for
/// primality; the Carmichael numbers show that the converse restricted to
/// coprime bases is false.

from nat import Nat, has_prime_divisor, divides_lte, divides_trans, lte_trans,
    lte_mul_both, lte_mul, lt_or_lte, lte_antisymm, mul_cancel_left,
    mul_one_right, mul_comm
from number_theory.fermat import fermat_euler
from number_theory.fermat_consequences import fermat_base_two_odd_prime
from number_theory.carmichael import carmichael_witness, carmichael_witness_apply
from number_theory.carmichael_properties import is_carmichael_number,
    is_carmichael_number_apply, is_carmichael_number_561
from number_theory.tau_multiplicative import nat_tau, nat_tau_two_iff_prime,
    lte_neq_imp_lt
from number_theory.factorisation import prime_divisor_is_one_or_self
from number_theory.totient import not_coprime_imp_divides_prime
numerals Nat

// ---------------------------------------------------------------------------
// The Fermat test.
// ---------------------------------------------------------------------------

/// The Fermat test: for a prime `p` and a base `a` coprime to `p`, the power
/// `a^(p - 1)` is congruent to `1` modulo `p`. Coprimality with `p` means
/// `gcd(a, p) = 1`. This is `fermat_euler` from fermat.ac, restated.
theorem fermat_test_prime(p: Nat, a: Nat) {
    p.is_prime and a.coprime(p) implies a.pow(p - Nat.1).congr_mod(Nat.1, p)
} by {
    if p.is_prime and a.coprime(p) {
        fermat_euler(p, a)
        a.pow(p - Nat.1).congr_mod(Nat.1, p)
    }
}

/// The Fermat test with the coprimality hypothesis written as a unit gcd:
/// for a prime `p` and a base `a` with `gcd(a, p) = 1`, the power
/// `a^(p - 1)` is congruent to `1` modulo `p`.
theorem fermat_test_prime_gcd(p: Nat, a: Nat) {
    p.is_prime and a.gcd(p) = Nat.1 implies a.pow(p - Nat.1).congr_mod(Nat.1, p)
} by {
    if p.is_prime and a.gcd(p) = Nat.1 {
        a.coprime(p)
        fermat_euler(p, a)
        a.pow(p - Nat.1).congr_mod(Nat.1, p)
    }
}

/// The Fermat test for bases not divisible by `p`: for a prime `p` and a
/// base `a` with `p ∤ a`, the power `a^(p - 1)` is congruent to `1` modulo
/// `p`. A prime fails to divide `a` exactly when `a` is coprime to `p`, so
/// this is the previous statement rephrased.
theorem fermat_test_prime_not_dividing(p: Nat, a: Nat) {
    p.is_prime and not p.divides(a) implies a.pow(p - Nat.1).congr_mod(Nat.1, p)
} by {
    if p.is_prime and not p.divides(a) {
        if not a.coprime(p) {
            not_coprime_imp_divides_prime(p, a)
            p.divides(a)
            false
        }
        a.coprime(p)
        fermat_euler(p, a)
        a.pow(p - Nat.1).congr_mod(Nat.1, p)
    }
}

// ---------------------------------------------------------------------------
// Carmichael numbers: the Fermat test converse is false.
// ---------------------------------------------------------------------------

/// The defining predicate of a Carmichael number, restated: `n` is a
/// Carmichael number exactly when `n` is composite and `n - 1` is a
/// universal exponent modulo `n` (restated from `is_carmichael_number_apply`
/// in carmichael_properties.ac).
theorem carmichael_number_apply(n: Nat) {
    is_carmichael_number(n) =
        (n.is_composite and carmichael_witness(n)(n - Nat.1))
} by {
    is_carmichael_number_apply(n)
    is_carmichael_number(n) =
        (n.is_composite and carmichael_witness(n)(n - Nat.1))
}

/// `561` is a Carmichael number: the composite number `561` satisfies
/// `a^560 ≡ 1 (mod 561)` for every base `a` coprime to `561` (restated from
/// `is_carmichael_number_561` in carmichael_properties.ac).
theorem carmichael_561 {
    is_carmichael_number(Nat.561)
} by {
    is_carmichael_number_561
    is_carmichael_number(Nat.561)
}

/// The converse of the Fermat test is false for coprime bases: there exists
/// a composite `n` such that `a^(n - 1) ≡ 1 (mod n)` for every base `a`
/// coprime to `n`. The witness is `n = 561`, a Carmichael number.
theorem fermat_converse_false_for_coprime_bases {
    exists(n: Nat) {
        n.is_composite and forall(a: Nat) {
            a.coprime(n) implies a.pow(n - Nat.1).congr_mod(Nat.1, n)
        }
    }
} by {
    is_carmichael_number_561
    is_carmichael_number(Nat.561)
    is_carmichael_number_apply(Nat.561)
    is_carmichael_number(Nat.561) =
        (Nat.561.is_composite and carmichael_witness(Nat.561)(Nat.561 - Nat.1))
    Nat.561.is_composite and carmichael_witness(Nat.561)(Nat.561 - Nat.1)
    carmichael_witness(Nat.561)(Nat.561 - Nat.1)
    carmichael_witness_apply(Nat.561, Nat.561 - Nat.1)
    carmichael_witness(Nat.561)(Nat.561 - Nat.1) =
        (Nat.0 < Nat.561 - Nat.1 and forall(a: Nat) {
            a.coprime(Nat.561) implies a.pow(Nat.561 - Nat.1).congr_mod(Nat.1, Nat.561)
        })
    forall(a: Nat) {
        a.coprime(Nat.561) implies a.pow(Nat.561 - Nat.1).congr_mod(Nat.1, Nat.561)
    }
    Nat.561.is_composite
    Nat.561.is_composite and forall(a: Nat) {
        a.coprime(Nat.561) implies a.pow(Nat.561 - Nat.1).congr_mod(Nat.1, Nat.561)
    }
    exists(n: Nat) {
        n.is_composite and forall(a: Nat) {
            a.coprime(n) implies a.pow(n - Nat.1).congr_mod(Nat.1, n)
        }
    }
}

// ---------------------------------------------------------------------------
// The base-2 Fermat test.
// ---------------------------------------------------------------------------

/// The base-2 Fermat test: for an odd prime `n` (that is, `n` prime and
/// `n != 2`), the power `2^(n - 1)` is congruent to `1` modulo `n`. The
/// case `n = 2` is excluded, since `2^(2 - 1) = 2 ≢ 1 (mod 2)`. This is
/// `fermat_base_two_odd_prime` from fermat_consequences.ac, restated for
/// odd primes.
theorem fermat_test_base_two(n: Nat) {
    n.is_prime and n != Nat.2 implies Nat.2.pow(n - Nat.1).congr_mod(Nat.1, n)
} by {
    if n.is_prime and n != Nat.2 {
        Nat.1 < n
        Nat.2 <= n
        lte_neq_imp_lt(Nat.2, n)
        Nat.2 < n
        fermat_base_two_odd_prime(n)
        Nat.2.pow(n - Nat.1).congr_mod(Nat.1, n)
    }
}

// ---------------------------------------------------------------------------
// The two-divisor test.
// ---------------------------------------------------------------------------

/// The two-divisor test: `n` is prime exactly when it has exactly two
/// positive divisors, i.e. `τ(n) = 2`, where `τ` counts the positive
/// divisors (restated from `nat_tau_two_iff_prime` in
/// tau_multiplicative.ac).
theorem exactly_two_divisors_iff_prime(n: Nat) {
    (nat_tau(n) = Nat.2) = n.is_prime
} by {
    nat_tau_two_iff_prime(n)
    (nat_tau(n) = Nat.2) = n.is_prime
}

// ---------------------------------------------------------------------------
// Trial division.
// ---------------------------------------------------------------------------

/// If a number greater than one has a prime divisor, then it has a prime
/// divisor at most its square root, in the Nat-friendly form `p * p <= a`:
/// a prime divisor of `a` is at most `a`, so its square is at most `a * a`.
theorem small_factor_imp_small_prime(a: Nat, n: Nat) {
    Nat.1 < a and a * a <= n and a.divides(n)
        implies exists(p: Nat) { p.is_prime and p.divides(n) and p * p <= n }
} by {
    if Nat.1 < a and a * a <= n and a.divides(n) {
        has_prime_divisor(a)
        let p: Nat satisfy { p.is_prime and p.divides(a) }
        p.is_prime
        divides_trans(p, a, n)
        p.divides(n)
        divides_lte(p, a)
        a != Nat.0
        p <= a
        lte_mul_both(p, p, a)
        p * p <= p * a
        lte_mul_both(a, p, a)
        a * p <= a * a
        mul_comm(p, a)
        p * a = a * p
        p * p <= a * p
        lte_trans(p * p, a * p, a * a)
        p * p <= a * a
        lte_trans(p * p, a * a, n)
        p * p <= n
        p.is_prime and p.divides(n) and p * p <= n
        exists(q: Nat) { q.is_prime and q.divides(n) and q * q <= n }
    }
}

/// The classical direction of trial division: a composite `n = b * c` has a
/// prime factor `p` with `p * p <= n`, i.e. `p <= sqrt(n)`. The smaller of
/// the two factors `b`, `c` squares to at most `n`, and any prime divisor of
/// that factor is at most it.
theorem composite_imp_small_prime_factor(n: Nat) {
    n.is_composite implies exists(p: Nat) {
        p.is_prime and p.divides(n) and p * p <= n
    }
} by {
    if n.is_composite {
        let (b: Nat, c: Nat) satisfy {
            Nat.1 < b and Nat.1 < c and n = b * c
        }
        lt_or_lte(c, b)
        if c < b {
            c <= b
            lte_mul_both(c, c, b)
            c * c <= c * b
            mul_comm(c, b)
            c * b = b * c
            b * c = n
            c * b = n
            c * c <= n
            c.divides(n)
            Nat.1 < c and c * c <= n and c.divides(n)
            small_factor_imp_small_prime(c, n)
            exists(p: Nat) { p.is_prime and p.divides(n) and p * p <= n }
        } else {
            b <= c
            lte_mul_both(b, b, c)
            b * b <= b * c
            b * c = n
            b * b <= n
            b.divides(n)
            Nat.1 < b and b * b <= n and b.divides(n)
            small_factor_imp_small_prime(b, n)
            exists(p: Nat) { p.is_prime and p.divides(n) and p * p <= n }
        }
    }
}

/// The easy direction of trial division: a prime has no prime divisor `p`
/// with `p * p <= n`. A prime divisor of the prime `n` is `1` or `n`
/// itself; it cannot be `1`, and `p = n` contradicts `p * p <= n`.
theorem prime_imp_no_small_prime_divisor(n: Nat) {
    n.is_prime implies forall(p: Nat) {
        p.is_prime and p * p <= n implies not p.divides(n)
    }
} by {
    if n.is_prime {
        forall(p: Nat) {
            if p.is_prime and p * p <= n {
                if p.divides(n) {
                    prime_divisor_is_one_or_self(n, p)
                    p = Nat.1 or p = n
                    Nat.1 < p
                    p != Nat.1
                    p = n
                    p * p = n * n
                    n * n <= n
                    lte_mul(n, n)
                    n != Nat.0
                    n <= n * n
                    lte_antisymm(n * n, n)
                    n * n = n
                    mul_one_right(n)
                    n * Nat.1 = n
                    n * n = n * Nat.1
                    mul_cancel_left(n, n, Nat.1)
                    n = Nat.1
                    Nat.1 < n
                    false
                }
                not p.divides(n)
            }
        }
    }
}

/// Trial division: `n` is prime exactly when `n > 1` and no prime `p` with
/// `p * p <= n` (i.e. `p <= sqrt(n)`) divides `n`. The "if" direction is the
/// classical argument: a composite `n = b * c` has a prime factor at most
/// `sqrt(n)`, and the "only if" direction holds because a prime divisor of a
/// prime is that prime itself.
theorem trial_division_test(n: Nat) {
    n.is_prime = (Nat.1 < n and forall(p: Nat) {
        p.is_prime and p * p <= n implies not p.divides(n)
    })
} by {
    if n.is_prime {
        Nat.1 < n
        prime_imp_no_small_prime_divisor(n)
        forall(p: Nat) {
            p.is_prime and p * p <= n implies not p.divides(n)
        }
        Nat.1 < n and forall(p: Nat) {
            p.is_prime and p * p <= n implies not p.divides(n)
        }
    }
    if Nat.1 < n and forall(p: Nat) {
        p.is_prime and p * p <= n implies not p.divides(n)
    } {
        if not n.is_prime {
            n.is_composite
            composite_imp_small_prime_factor(n)
            let p: Nat satisfy { p.is_prime and p.divides(n) and p * p <= n }
            p.is_prime and p * p <= n
            forall(q: Nat) {
                q.is_prime and q * q <= n implies not q.divides(n)
            }
            not p.divides(n)
            p.divides(n)
            false
        }
        n.is_prime
    }
}
