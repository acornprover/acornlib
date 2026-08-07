from nat import Nat
from nat import mod_mod, add_mod, lte_trans
from nat import exp_zero
from nat import div_sub_mod, mod_of_zero, sub_zero, div_imp_mod
numerals Nat

attributes Nat {
    /// True if this number is congruent to b modulo n. Defined as the equality
    /// of the two values' remainders when divided by n. This avoids the
    /// truncating subtraction on Nat that the divisibility-based form would
    /// require.
    define congr_mod(self, b: Nat, n: Nat) -> Bool {
        self.mod(n) = b.mod(n)
    }
}

/// Congruence modulo n is reflexive.
theorem congr_mod_refl(a: Nat, n: Nat) {
    a.congr_mod(a, n)
}

/// Congruence modulo n is symmetric.
theorem congr_mod_symm(a: Nat, b: Nat, n: Nat) {
    a.congr_mod(b, n) implies b.congr_mod(a, n)
}

/// Congruence modulo n is transitive.
theorem congr_mod_trans(a: Nat, b: Nat, c: Nat, n: Nat) {
    a.congr_mod(b, n) and b.congr_mod(c, n) implies a.congr_mod(c, n)
} by {
    if a.congr_mod(b, n) and b.congr_mod(c, n) {
        b.mod(n) = c.mod(n)
        a.mod(n) = c.mod(n)
    }
}

/// A natural is congruent to its remainder modulo n.
theorem mod_congr_mod_self(a: Nat, n: Nat) {
    a.mod(n).congr_mod(a, n)
} by {
    mod_mod(a, n)
}

/// Helper: a positive natural multiplied by a positive natural is at least the second.
theorem mul_geq_when_pos(d: Nat, n: Nat) {
    d != Nat.0 implies n <= d * n
} by {
    if d != Nat.0 {
        let k: Nat satisfy { d = k.suc }
        k.suc * n = k * n + n
        n <= k * n + n
        n <= d * n
    }
}

/// Quotient-remainder uniqueness: when q1*n + r1 = q2*n + r2 with both
/// remainders below n, the remainders must coincide.
theorem mod_unique(q1: Nat, q2: Nat, r1: Nat, r2: Nat, n: Nat) {
    n != Nat.0 and q1 * n + r1 = q2 * n + r2 and r1 < n and r2 < n
        implies r1 = r2
} by {
    if n != Nat.0 and q1 * n + r1 = q2 * n + r2 and r1 < n and r2 < n {
        if q1 = q2 {
        } else {
            if q1 < q2 {
                let d: Nat satisfy { q1 + d = q2 }
                d != Nat.0
                mul_geq_when_pos(d, n)
                n <= d * n
                q1 * n + r1 = (q1 + d) * n + r2
                q1 * n + r1 = q1 * n + d * n + r2
                r1 = d * n + r2
                d * n <= r1
                n <= r1
                false
            } else {
                q2 < q1
                let d: Nat satisfy { q2 + d = q1 }
                d != Nat.0
                mul_geq_when_pos(d, n)
                n <= d * n
                (q2 + d) * n + r1 = q2 * n + r2
                q2 * n + d * n + r1 = q2 * n + r2
                d * n + r1 = r2
                d * n <= r2
                n <= r2
                false
            }
        }
    }
}

/// The mod result is strictly less than the modulus when the modulus is nonzero.
theorem mod_lt(a: Nat, n: Nat) {
    n != Nat.0 implies a.mod(n) < n
}

/// Adding a multiple of n leaves the remainder unchanged.
theorem mod_add_mul(q: Nat, n: Nat, r: Nat) {
    (q * n + r).mod(n) = r.mod(n)
} by {
    if n = Nat.0 {
        Nat.0 + r = r
    } else {
        let r1: Nat = (q * n + r).mod(n)
        let r2: Nat = r.mod(n)
        mod_lt(q * n + r, n)
        mod_lt(r, n)
        r2 < n
        add_mod(q * n + r, n)
        let big_q: Nat satisfy { big_q * n + r1 = q * n + r }
        add_mod(r, n)
        let small_q: Nat satisfy { small_q * n + r2 = r }
        let other_q: Nat = q + small_q
        q * n + small_q * n + r2 = q * n + r
        big_q * n + r1 = other_q * n + r2
        mod_unique(big_q, other_q, r1, r2, n)
        r1 = r2
        (q * n + r).mod(n) = r.mod(n)
    }
}

/// Modular addition reduces componentwise: the remainder of a + b modulo n
/// only depends on the remainders of a and b.
theorem mod_add_eq(a: Nat, b: Nat, n: Nat) {
    (a + b).mod(n) = (a.mod(n) + b.mod(n)).mod(n)
} by {
    if n = Nat.0 {
        a + b = a.mod(Nat.0) + b.mod(Nat.0)
    } else {
        add_mod(a, n)
        let qa: Nat satisfy { qa * n + a.mod(n) = a }
        add_mod(b, n)
        let qb: Nat satisfy { qb * n + b.mod(n) = b }
        let s: Nat = qa + qb
        let r: Nat = a.mod(n) + b.mod(n)
        qa * n + a.mod(n) + (qb * n + b.mod(n)) = a + b
        qa * n + qb * n + a.mod(n) + b.mod(n) = a + b
        s * n + r = a + b
        mod_add_mul(s, n, r)
        (a + b).mod(n) = (a.mod(n) + b.mod(n)).mod(n)
    }
}

/// Replacing a factor by its remainder leaves the product's remainder unchanged.
theorem mod_mul_left_eq(a: Nat, b: Nat, n: Nat) {
    (a * b).mod(n) = (a.mod(n) * b).mod(n)
} by {
    if n = Nat.0 {
        a.mod(Nat.0) = a
    } else {
        add_mod(a, n)
        let qa: Nat satisfy { qa * n + a.mod(n) = a }
        (qa * n + a.mod(n)) * b = qa * n * b + a.mod(n) * b
        qa * (b * n) = qa * b * n
        mod_add_mul(qa * b, n, a.mod(n) * b)
        (a * b).mod(n) = (a.mod(n) * b).mod(n)
    }
}

/// Replacing a factor on the right by its remainder leaves the product's remainder unchanged.
theorem mod_mul_right_eq(a: Nat, b: Nat, n: Nat) {
    (a * b).mod(n) = (a * b.mod(n)).mod(n)
} by {
    mod_mul_left_eq(b, a, n)
}

/// Modular multiplication reduces componentwise: the remainder of a * b
/// modulo n only depends on the remainders of a and b.
theorem mod_mul_eq(a: Nat, b: Nat, n: Nat) {
    (a * b).mod(n) = (a.mod(n) * b.mod(n)).mod(n)
} by {
    mod_mul_left_eq(a, b, n)
    mod_mul_right_eq(a.mod(n), b, n)
}

/// Congruence modulo n is preserved by addition.
theorem congr_mod_add(a: Nat, b: Nat, c: Nat, d: Nat, n: Nat) {
    a.congr_mod(c, n) and b.congr_mod(d, n) implies (a + b).congr_mod(c + d, n)
} by {
    if a.congr_mod(c, n) and b.congr_mod(d, n) {
        a.mod(n) = c.mod(n)
        b.mod(n) = d.mod(n)
        mod_add_eq(a, b, n)
        mod_add_eq(c, d, n)
        (a + b).mod(n) = (c + d).mod(n)
    }
}

/// Congruence modulo n is preserved by multiplication.
theorem congr_mod_mul(a: Nat, b: Nat, c: Nat, d: Nat, n: Nat) {
    a.congr_mod(c, n) and b.congr_mod(d, n) implies (a * b).congr_mod(c * d, n)
} by {
    if a.congr_mod(c, n) and b.congr_mod(d, n) {
        a.mod(n) = c.mod(n)
        b.mod(n) = d.mod(n)
        mod_mul_eq(a, b, n)
        mod_mul_eq(c, d, n)
        (a * b).mod(n) = (c * d).mod(n)
    }
}

/// Helper for the power-preservation induction: only the inductive
/// implication, with the congruence hypothesis externalised.
theorem congr_mod_pow_step(a: Nat, b: Nat, n: Nat, x: Nat) {
    a.congr_mod(b, n) and a.pow(x).congr_mod(b.pow(x), n)
        implies a.pow(x.suc).congr_mod(b.pow(x.suc), n)
} by {
    if a.congr_mod(b, n) and a.pow(x).congr_mod(b.pow(x), n) {
        congr_mod_mul(a.pow(x), a, b.pow(x), b, n)
        a.pow(x.suc) = a.pow(x) * a
        a.pow(x.suc).congr_mod(b.pow(x) * b, n)
        a.pow(x.suc).congr_mod(b.pow(x.suc), n)
    }
}

/// Congruence modulo n is preserved by raising to a natural-number power.
theorem congr_mod_pow(a: Nat, b: Nat, n: Nat, k: Nat) {
    a.congr_mod(b, n) implies a.pow(k).congr_mod(b.pow(k), n)
} by {
    let f: Nat -> Bool = function(x: Nat) {
        a.pow(x).congr_mod(b.pow(x), n)
    }
    exp_zero(a)
    exp_zero(b)
    f(Nat.0)
    forall(x: Nat) {
        if f(x) {
            congr_mod_pow_step(a, b, n, x)
            f(x.suc)
        }
    }
}

/// Divisibility by d gives congruence to zero modulo d.
theorem congr_mod_zero_of_divides(d: Nat, x: Nat) {
    d.divides(x) implies x.congr_mod(Nat.0, d)
} by {
    if d.divides(x) {
        div_imp_mod(x, d)
        x.mod(d) = Nat.0
        Nat.0.mod(d) = Nat.0
        x.mod(d) = Nat.0.mod(d)
    }
}

/// Congruence to zero modulo d gives divisibility by d.
theorem divides_of_congr_mod_zero(d: Nat, x: Nat) {
    x.congr_mod(Nat.0, d) implies d.divides(x)
} by {
    if x.congr_mod(Nat.0, d) {
        x.mod(d) = Nat.0.mod(d)
        mod_of_zero(d)
        Nat.0.mod(d) = Nat.0
        x.mod(d) = Nat.0
        div_sub_mod(x, d)
        d.divides(x - x.mod(d))
        x - x.mod(d) = x - Nat.0
        sub_zero(x)
        x - Nat.0 = x
        d.divides(x)
    }
}

/// A divisibility relation d | x is equivalent to x being congruent to zero
/// modulo d.
theorem congr_mod_zero_iff_divides(d: Nat, x: Nat) {
    x.congr_mod(Nat.0, d) = d.divides(x)
} by {
    if x.congr_mod(Nat.0, d) {
        divides_of_congr_mod_zero(d, x)
    }
    if d.divides(x) {
        congr_mod_zero_of_divides(d, x)
    }
}
