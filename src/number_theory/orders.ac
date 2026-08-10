/// Orders of elements modulo `n` and generators of the unit group.
///
/// The multiplicative order of `a` modulo `n` — the least positive `k` with
/// `a^k ≡ 1 (mod n)` — is formalised in `multiplicative_order.ac`:
/// `is_multiplicative_order_mod(a, n, k)` is the least-positive-exponent
/// predicate and `multiplicative_order_mod(a, n)` is the selected value
/// (with placeholder `0` outside the positive-coprime domain). This file
/// restates the order facts needed for orders and generators, records the
/// divisibility of the order into Euler's totient, verifies the classical
/// period law `a^k ≡ 1 (mod n)` iff `ord_n(a) | k`, and computes the orders
/// of `2` modulo `5` and `3` modulo `7` (which is a primitive root modulo
/// `7`). The classical count `φ(p - 1)` of primitive roots modulo `p` is
/// stated and verified for `p = 5`.

from number_theory.multiplicative_order import Nat, is_multiplicative_order_mod,
    multiplicative_order_mod, multiplicative_order_mod_is_order,
    multiplicative_order_mod_minimal, multiplicative_order_mod_positive,
    multiplicative_order_pow_congr_one, multiplicative_order_mod_divides_exponent_iff
from number_theory.modular_applications import order_divides_totient
from number_theory.fermat_consequences import order_mod_prime_divides_pred
from number_theory.primitive_root_applications import order_two_mod_five,
    two_coprime_mod_five, three_coprime_mod_five, order_three_mod_five,
    order_one_mod_five, order_four_mod_five, zero_not_coprime_mod_five,
    pow_two_one_mod_five, pow_two_two_mod_five, pow_two_three_mod_five,
    mod_three_mod_five, lt_three_mod_five,
    congr_two_pow_four_mod_five, order_four_pred_mod_five,
    elements_of_order_four_count_mod_five, totient_four_mod_five
from number_theory.primitive_root_applications2 import is_primitive_root_mod,
    two_is_primitive_root_mod_five, two_ne_one, three_ne_one, four_ne_one,
    five_ne_one, six_ne_one, one_plus_three, two_plus_two, two_plus_three,
    four_plus_one, lt_four_seven, lt_five_seven, lt_six_seven, lt_zero_six
from number_theory.zsigmondy import five_is_prime, seven_is_prime, lt_one_seven,
    lt_three_seven, three_pow_two, lt_ne
from number_theory.totient import coprime_below_prime
from number_theory.congruence import congr_mod_refl, congr_mod_symm,
    congr_mod_trans, congr_mod_mul, congr_mod_pow
from data.basic.logic import eq_true_intro, eq_false_intro
from list import List
from nat import exp_one, exp_add, exp_mul, one_exp, small_mod, mod_of_decomp,
    lt_suc, lt_imp_lt_suc, lt_suc_right, lt_not_ref, not_lt_zero, trichotomy,
    lte_imp_not_lt, add_imp_sub, add_one_right, divides_self,
    nat_add_7_2, nat_mul_1_5, nat_mul_1_7, nat_mul_2_2, nat_mul_2_3,
    nat_mul_2_6, nat_mul_4_2, nat_mul_5_7, nat_mul_6_6
numerals Nat

// ---------------------------------------------------------------------------
// The order of an element modulo n.
// ---------------------------------------------------------------------------

/// On the positive-coprime domain, the selected multiplicative order is the
/// least positive exponent making the power congruent to `1`.
theorem order_is_least_positive_exponent(a: Nat, n: Nat) {
    n != Nat.0 and a.coprime(n) implies
        is_multiplicative_order_mod(a, n, multiplicative_order_mod(a, n))
} by {
    if n != Nat.0 and a.coprime(n) {
        multiplicative_order_mod_is_order(a, n)
        is_multiplicative_order_mod(a, n, multiplicative_order_mod(a, n))
    }
}

/// The selected order is minimal among positive exponents giving congruence
/// to `1`.
theorem order_least_positive_exponent(a: Nat, n: Nat, j: Nat) {
    n != Nat.0 and a.coprime(n) and Nat.0 < j and a.pow(j).congr_mod(Nat.1, n)
        implies multiplicative_order_mod(a, n) <= j
} by {
    if n != Nat.0 and a.coprime(n) and Nat.0 < j and a.pow(j).congr_mod(Nat.1, n) {
        multiplicative_order_mod_minimal(a, n, j)
        multiplicative_order_mod(a, n) <= j
    }
}

// ---------------------------------------------------------------------------
// The order divides Euler's totient.
//
// The general divisibility `ord_n(a) | φ(n)` for `gcd(a, n) = 1` is proved
// in modular_applications.ac as `order_divides_totient`; the prime form
// `ord_p(a) | p - 1` is proved in fermat_consequences.ac as
// `order_mod_prime_divides_pred`. Both are restated here.
// ---------------------------------------------------------------------------

/// The order of `a` modulo `n` divides Euler's totient `φ(n)`, for `a`
/// coprime to `n`.
theorem order_divides_euler_totient(a: Nat, n: Nat) {
    n != Nat.0 and a.coprime(n)
        implies multiplicative_order_mod(a, n).divides(n.totient)
} by {
    if n != Nat.0 and a.coprime(n) {
        order_divides_totient(a, n)
        multiplicative_order_mod(a, n).divides(n.totient)
    }
}

/// The order of `a` modulo the prime `p` divides `p - 1`, for `a` coprime
/// to `p`: since `φ(p) = p - 1` this is the totient divisibility.
theorem order_divides_prime_minus_one(a: Nat, p: Nat) {
    p.is_prime and a.coprime(p)
        implies multiplicative_order_mod(a, p).divides(p - Nat.1)
} by {
    if p.is_prime and a.coprime(p) {
        order_mod_prime_divides_pred(p, a)
        multiplicative_order_mod(a, p).divides(p - Nat.1)
    }
}

// ---------------------------------------------------------------------------
// The order of 2 modulo 5.
// ---------------------------------------------------------------------------

/// `2` has multiplicative order `4` modulo `5`.
theorem order_two_mod_five_is_four {
    multiplicative_order_mod(Nat.2, Nat.5) = Nat.4
} by {
    order_two_mod_five
    multiplicative_order_mod(Nat.2, Nat.5) = Nat.4
}

/// `2^1 ≡ 2 (mod 5)`.
theorem congr_two_pow_one_mod_five {
    Nat.2.pow(Nat.1).congr_mod(Nat.2, Nat.5)
} by {
    pow_two_one_mod_five
    Nat.2.pow(Nat.1) = Nat.2
    congr_mod_refl(Nat.2, Nat.5)
    Nat.2.congr_mod(Nat.2, Nat.5)
    Nat.2.pow(Nat.1).congr_mod(Nat.2, Nat.5)
}

/// `2^2 ≡ 4 (mod 5)`.
theorem congr_two_pow_two_mod_five {
    Nat.2.pow(Nat.2).congr_mod(Nat.4, Nat.5)
} by {
    pow_two_two_mod_five
    Nat.2.pow(Nat.2) = Nat.4
    congr_mod_refl(Nat.4, Nat.5)
    Nat.4.congr_mod(Nat.4, Nat.5)
    Nat.2.pow(Nat.2).congr_mod(Nat.4, Nat.5)
}

/// `2^3 ≡ 3 (mod 5)`.
theorem congr_two_pow_three_mod_five {
    Nat.2.pow(Nat.3).congr_mod(Nat.3, Nat.5)
} by {
    pow_two_three_mod_five
    Nat.2.pow(Nat.3) = Nat.8
    nat_mul_1_5
    Nat.1 * Nat.5 = Nat.5
    Nat.5 + Nat.3 = Nat.8
    lt_three_mod_five
    Nat.3 < Nat.5
    mod_of_decomp(Nat.1, Nat.3, Nat.5)
    (Nat.1 * Nat.5 + Nat.3).mod(Nat.5) = Nat.3
    Nat.8.mod(Nat.5) = Nat.3
    mod_three_mod_five
    Nat.3.mod(Nat.5) = Nat.3
    Nat.8.mod(Nat.5) = Nat.3.mod(Nat.5)
    Nat.8.congr_mod(Nat.3, Nat.5)
    Nat.2.pow(Nat.3).congr_mod(Nat.3, Nat.5)
}

/// The powers of `2` modulo `5`: `2^1 ≡ 2`, `2^2 ≡ 4`, `2^3 ≡ 3`, and
/// `2^4 ≡ 1`.
theorem two_powers_mod_five_table {
    Nat.2.pow(Nat.1).congr_mod(Nat.2, Nat.5) and
        Nat.2.pow(Nat.2).congr_mod(Nat.4, Nat.5) and
        Nat.2.pow(Nat.3).congr_mod(Nat.3, Nat.5) and
        Nat.2.pow(Nat.4).congr_mod(Nat.1, Nat.5)
} by {
    congr_two_pow_one_mod_five
    Nat.2.pow(Nat.1).congr_mod(Nat.2, Nat.5)
    congr_two_pow_two_mod_five
    Nat.2.pow(Nat.2).congr_mod(Nat.4, Nat.5)
    congr_two_pow_three_mod_five
    Nat.2.pow(Nat.3).congr_mod(Nat.3, Nat.5)
    congr_two_pow_four_mod_five
    Nat.2.pow(Nat.4).congr_mod(Nat.1, Nat.5)
    Nat.2.pow(Nat.1).congr_mod(Nat.2, Nat.5) and
        Nat.2.pow(Nat.2).congr_mod(Nat.4, Nat.5) and
        Nat.2.pow(Nat.3).congr_mod(Nat.3, Nat.5) and
        Nat.2.pow(Nat.4).congr_mod(Nat.1, Nat.5)
}

/// `2` is a primitive root modulo `5`: it is coprime to `5` and its order
/// `4` is the full order `φ(5) = 5 - 1`.
theorem two_is_primitive_root_mod_five_restated {
    is_primitive_root_mod(Nat.2, Nat.5)
} by {
    two_is_primitive_root_mod_five
    is_primitive_root_mod(Nat.2, Nat.5)
}

// ---------------------------------------------------------------------------
// The order of 3 modulo 7.
// ---------------------------------------------------------------------------

/// `2 < 7`.
theorem lt_two_seven {
    Nat.2 < Nat.7
} by {
    lt_suc(Nat.2)
    Nat.2 < Nat.3
    lt_imp_lt_suc(Nat.2, Nat.3)
    Nat.2 < Nat.4
    lt_imp_lt_suc(Nat.2, Nat.4)
    Nat.2 < Nat.5
    lt_imp_lt_suc(Nat.2, Nat.5)
    Nat.2 < Nat.6
    lt_imp_lt_suc(Nat.2, Nat.6)
    Nat.2 < Nat.7
}

/// `3` is coprime to `7`.
theorem three_coprime_mod_seven {
    Nat.3.coprime(Nat.7)
} by {
    seven_is_prime
    Nat.7.is_prime
    Nat.1 <= Nat.3
    lt_three_seven
    Nat.3 < Nat.7
    coprime_below_prime(Nat.7, Nat.3)
    Nat.3.coprime(Nat.7)
}

/// `1.mod(7) = 1`.
theorem mod_one_mod_seven {
    Nat.1.mod(Nat.7) = Nat.1
} by {
    lt_one_seven
    Nat.1 < Nat.7
    small_mod(Nat.1, Nat.7)
}

/// `2.mod(7) = 2`.
theorem mod_two_mod_seven {
    Nat.2.mod(Nat.7) = Nat.2
} by {
    lt_two_seven
    Nat.2 < Nat.7
    small_mod(Nat.2, Nat.7)
}

/// `3.mod(7) = 3`.
theorem mod_three_mod_seven {
    Nat.3.mod(Nat.7) = Nat.3
} by {
    lt_three_seven
    Nat.3 < Nat.7
    small_mod(Nat.3, Nat.7)
}

/// `4.mod(7) = 4`.
theorem mod_four_mod_seven {
    Nat.4.mod(Nat.7) = Nat.4
} by {
    lt_four_seven
    Nat.4 < Nat.7
    small_mod(Nat.4, Nat.7)
}

/// `5.mod(7) = 5`.
theorem mod_five_mod_seven {
    Nat.5.mod(Nat.7) = Nat.5
} by {
    lt_five_seven
    Nat.5 < Nat.7
    small_mod(Nat.5, Nat.7)
}

/// `6.mod(7) = 6`.
theorem mod_six_mod_seven {
    Nat.6.mod(Nat.7) = Nat.6
} by {
    lt_six_seven
    Nat.6 < Nat.7
    small_mod(Nat.6, Nat.7)
}

/// `2 ≢ 1 (mod 7)`.
theorem not_congr_two_mod_seven {
    not Nat.2.congr_mod(Nat.1, Nat.7)
} by {
    mod_two_mod_seven
    Nat.2.mod(Nat.7) = Nat.2
    mod_one_mod_seven
    Nat.1.mod(Nat.7) = Nat.1
    if Nat.2.congr_mod(Nat.1, Nat.7) {
        Nat.2.mod(Nat.7) = Nat.1.mod(Nat.7)
        Nat.2 = Nat.1
        two_ne_one
        false
    }
}

/// `3 ≢ 1 (mod 7)`.
theorem not_congr_three_mod_seven {
    not Nat.3.congr_mod(Nat.1, Nat.7)
} by {
    mod_three_mod_seven
    Nat.3.mod(Nat.7) = Nat.3
    mod_one_mod_seven
    Nat.1.mod(Nat.7) = Nat.1
    if Nat.3.congr_mod(Nat.1, Nat.7) {
        Nat.3.mod(Nat.7) = Nat.1.mod(Nat.7)
        Nat.3 = Nat.1
        three_ne_one
        false
    }
}

/// `4 ≢ 1 (mod 7)`.
theorem not_congr_four_mod_seven {
    not Nat.4.congr_mod(Nat.1, Nat.7)
} by {
    mod_four_mod_seven
    Nat.4.mod(Nat.7) = Nat.4
    mod_one_mod_seven
    Nat.1.mod(Nat.7) = Nat.1
    if Nat.4.congr_mod(Nat.1, Nat.7) {
        Nat.4.mod(Nat.7) = Nat.1.mod(Nat.7)
        Nat.4 = Nat.1
        four_ne_one
        false
    }
}

/// `5 ≢ 1 (mod 7)`.
theorem not_congr_five_mod_seven {
    not Nat.5.congr_mod(Nat.1, Nat.7)
} by {
    mod_five_mod_seven
    Nat.5.mod(Nat.7) = Nat.5
    mod_one_mod_seven
    Nat.1.mod(Nat.7) = Nat.1
    if Nat.5.congr_mod(Nat.1, Nat.7) {
        Nat.5.mod(Nat.7) = Nat.1.mod(Nat.7)
        Nat.5 = Nat.1
        five_ne_one
        false
    }
}

/// `6 ≢ 1 (mod 7)`.
theorem not_congr_six_mod_seven {
    not Nat.6.congr_mod(Nat.1, Nat.7)
} by {
    mod_six_mod_seven
    Nat.6.mod(Nat.7) = Nat.6
    mod_one_mod_seven
    Nat.1.mod(Nat.7) = Nat.1
    if Nat.6.congr_mod(Nat.1, Nat.7) {
        Nat.6.mod(Nat.7) = Nat.1.mod(Nat.7)
        Nat.6 = Nat.1
        six_ne_one
        false
    }
}

/// `9 ≡ 2 (mod 7)`.
theorem congr_nine_mod_seven {
    Nat.9.congr_mod(Nat.2, Nat.7)
} by {
    // 9 = 1 * 7 + 2, so 9 mod 7 = 2.
    nat_mul_1_7
    Nat.1 * Nat.7 = Nat.7
    nat_add_7_2
    Nat.7 + Nat.2 = Nat.9
    lt_two_seven
    Nat.2 < Nat.7
    mod_of_decomp(Nat.1, Nat.2, Nat.7)
    (Nat.1 * Nat.7 + Nat.2).mod(Nat.7) = Nat.2
    Nat.9.mod(Nat.7) = Nat.2
    mod_two_mod_seven
    Nat.2.mod(Nat.7) = Nat.2
    Nat.9.mod(Nat.7) = Nat.2.mod(Nat.7)
    Nat.9.congr_mod(Nat.2, Nat.7)
}

/// `12 ≡ 5 (mod 7)`.
theorem congr_twelve_mod_seven {
    Nat.12.congr_mod(Nat.5, Nat.7)
} by {
    // 12 = 1 * 7 + 5, so 12 mod 7 = 5.
    nat_mul_1_7
    Nat.1 * Nat.7 = Nat.7
    Nat.7 + Nat.5 = Nat.12
    lt_five_seven
    Nat.5 < Nat.7
    mod_of_decomp(Nat.1, Nat.5, Nat.7)
    (Nat.1 * Nat.7 + Nat.5).mod(Nat.7) = Nat.5
    Nat.12.mod(Nat.7) = Nat.5
    mod_five_mod_seven
    Nat.5.mod(Nat.7) = Nat.5
    Nat.12.mod(Nat.7) = Nat.5.mod(Nat.7)
    Nat.12.congr_mod(Nat.5, Nat.7)
}

/// `36 ≡ 1 (mod 7)`.
theorem congr_thirty_six_mod_seven {
    Nat.36.congr_mod(Nat.1, Nat.7)
} by {
    // 36 = 5 * 7 + 1, so 36 mod 7 = 1.
    nat_mul_5_7
    Nat.5 * Nat.7 = Nat.35
    Nat.35 + Nat.1 = Nat.36
    lt_one_seven
    Nat.1 < Nat.7
    mod_of_decomp(Nat.5, Nat.1, Nat.7)
    (Nat.5 * Nat.7 + Nat.1).mod(Nat.7) = Nat.1
    Nat.36.mod(Nat.7) = Nat.1
    mod_one_mod_seven
    Nat.1.mod(Nat.7) = Nat.1
    Nat.36.mod(Nat.7) = Nat.1.mod(Nat.7)
    Nat.36.congr_mod(Nat.1, Nat.7)
}

/// `3^1 ≡ 3 (mod 7)`.
theorem congr_three_pow_one_mod_seven {
    Nat.3.pow(Nat.1).congr_mod(Nat.3, Nat.7)
} by {
    exp_one(Nat.3)
    Nat.3.pow(Nat.1) = Nat.3
    congr_mod_refl(Nat.3, Nat.7)
    Nat.3.congr_mod(Nat.3, Nat.7)
    Nat.3.pow(Nat.1).congr_mod(Nat.3, Nat.7)
}

/// `3^2 ≡ 2 (mod 7)`.
theorem congr_three_pow_two_mod_seven {
    Nat.3.pow(Nat.2).congr_mod(Nat.2, Nat.7)
} by {
    three_pow_two
    Nat.3.pow(Nat.2) = Nat.9
    congr_nine_mod_seven
    Nat.9.congr_mod(Nat.2, Nat.7)
    Nat.3.pow(Nat.2).congr_mod(Nat.2, Nat.7)
}

/// `3^3 ≡ 6 (mod 7)`.
theorem congr_three_pow_three_mod_seven {
    Nat.3.pow(Nat.3).congr_mod(Nat.6, Nat.7)
} by {
    exp_add(Nat.3, Nat.2, Nat.1)
    Nat.3.pow(Nat.2 + Nat.1) = Nat.3.pow(Nat.2) * Nat.3.pow(Nat.1)
    Nat.2 + Nat.1 = Nat.3
    Nat.3.pow(Nat.3) = Nat.3.pow(Nat.2) * Nat.3.pow(Nat.1)
    congr_three_pow_two_mod_seven
    Nat.3.pow(Nat.2).congr_mod(Nat.2, Nat.7)
    congr_three_pow_one_mod_seven
    Nat.3.pow(Nat.1).congr_mod(Nat.3, Nat.7)
    congr_mod_mul(Nat.3.pow(Nat.2), Nat.3.pow(Nat.1), Nat.2, Nat.3, Nat.7)
    (Nat.3.pow(Nat.2) * Nat.3.pow(Nat.1)).congr_mod(Nat.2 * Nat.3, Nat.7)
    Nat.3.pow(Nat.3).congr_mod(Nat.2 * Nat.3, Nat.7)
    nat_mul_2_3
    Nat.2 * Nat.3 = Nat.6
    Nat.3.pow(Nat.3).congr_mod(Nat.6, Nat.7)
}

/// `3^4 ≡ 4 (mod 7)`.
theorem congr_three_pow_four_mod_seven {
    Nat.3.pow(Nat.4).congr_mod(Nat.4, Nat.7)
} by {
    exp_add(Nat.3, Nat.2, Nat.2)
    Nat.3.pow(Nat.2 + Nat.2) = Nat.3.pow(Nat.2) * Nat.3.pow(Nat.2)
    Nat.2 + Nat.2 = Nat.4
    Nat.3.pow(Nat.4) = Nat.3.pow(Nat.2) * Nat.3.pow(Nat.2)
    congr_three_pow_two_mod_seven
    Nat.3.pow(Nat.2).congr_mod(Nat.2, Nat.7)
    congr_mod_mul(Nat.3.pow(Nat.2), Nat.3.pow(Nat.2), Nat.2, Nat.2, Nat.7)
    (Nat.3.pow(Nat.2) * Nat.3.pow(Nat.2)).congr_mod(Nat.2 * Nat.2, Nat.7)
    Nat.3.pow(Nat.4).congr_mod(Nat.2 * Nat.2, Nat.7)
    nat_mul_2_2
    Nat.2 * Nat.2 = Nat.4
    Nat.3.pow(Nat.4).congr_mod(Nat.4, Nat.7)
}

/// `3^5 ≡ 5 (mod 7)`.
theorem congr_three_pow_five_mod_seven {
    Nat.3.pow(Nat.5).congr_mod(Nat.5, Nat.7)
} by {
    exp_add(Nat.3, Nat.2, Nat.3)
    Nat.3.pow(Nat.2 + Nat.3) = Nat.3.pow(Nat.2) * Nat.3.pow(Nat.3)
    two_plus_three
    Nat.2 + Nat.3 = Nat.5
    Nat.3.pow(Nat.5) = Nat.3.pow(Nat.2) * Nat.3.pow(Nat.3)
    congr_three_pow_two_mod_seven
    Nat.3.pow(Nat.2).congr_mod(Nat.2, Nat.7)
    congr_three_pow_three_mod_seven
    Nat.3.pow(Nat.3).congr_mod(Nat.6, Nat.7)
    congr_mod_mul(Nat.3.pow(Nat.2), Nat.3.pow(Nat.3), Nat.2, Nat.6, Nat.7)
    (Nat.3.pow(Nat.2) * Nat.3.pow(Nat.3)).congr_mod(Nat.2 * Nat.6, Nat.7)
    Nat.3.pow(Nat.5).congr_mod(Nat.2 * Nat.6, Nat.7)
    nat_mul_2_6
    Nat.2 * Nat.6 = Nat.12
    Nat.3.pow(Nat.5).congr_mod(Nat.12, Nat.7)
    congr_twelve_mod_seven
    Nat.12.congr_mod(Nat.5, Nat.7)
    congr_mod_trans(Nat.3.pow(Nat.5), Nat.12, Nat.5, Nat.7)
    Nat.3.pow(Nat.5).congr_mod(Nat.5, Nat.7)
}

/// `3^6 ≡ 1 (mod 7)`.
theorem congr_three_pow_six_mod_seven {
    Nat.3.pow(Nat.6).congr_mod(Nat.1, Nat.7)
} by {
    exp_add(Nat.3, Nat.3, Nat.3)
    Nat.3.pow(Nat.3 + Nat.3) = Nat.3.pow(Nat.3) * Nat.3.pow(Nat.3)
    Nat.3 + Nat.3 = Nat.6
    Nat.3.pow(Nat.6) = Nat.3.pow(Nat.3) * Nat.3.pow(Nat.3)
    congr_three_pow_three_mod_seven
    Nat.3.pow(Nat.3).congr_mod(Nat.6, Nat.7)
    congr_mod_mul(Nat.3.pow(Nat.3), Nat.3.pow(Nat.3), Nat.6, Nat.6, Nat.7)
    (Nat.3.pow(Nat.3) * Nat.3.pow(Nat.3)).congr_mod(Nat.6 * Nat.6, Nat.7)
    Nat.3.pow(Nat.6).congr_mod(Nat.6 * Nat.6, Nat.7)
    nat_mul_6_6
    Nat.6 * Nat.6 = Nat.36
    Nat.3.pow(Nat.6).congr_mod(Nat.36, Nat.7)
    congr_thirty_six_mod_seven
    Nat.36.congr_mod(Nat.1, Nat.7)
    congr_mod_trans(Nat.3.pow(Nat.6), Nat.36, Nat.1, Nat.7)
    Nat.3.pow(Nat.6).congr_mod(Nat.1, Nat.7)
}

/// `3^1 ≢ 1 (mod 7)`.
theorem not_congr_three_pow_one_mod_seven {
    not Nat.3.pow(Nat.1).congr_mod(Nat.1, Nat.7)
} by {
    exp_one(Nat.3)
    Nat.3.pow(Nat.1) = Nat.3
    not_congr_three_mod_seven
    not Nat.3.congr_mod(Nat.1, Nat.7)
    if Nat.3.pow(Nat.1).congr_mod(Nat.1, Nat.7) {
        Nat.3.congr_mod(Nat.1, Nat.7)
        false
    }
}

/// `3^2 ≢ 1 (mod 7)`.
theorem not_congr_three_pow_two_mod_seven {
    not Nat.3.pow(Nat.2).congr_mod(Nat.1, Nat.7)
} by {
    congr_three_pow_two_mod_seven
    Nat.3.pow(Nat.2).congr_mod(Nat.2, Nat.7)
    not_congr_two_mod_seven
    if Nat.3.pow(Nat.2).congr_mod(Nat.1, Nat.7) {
        congr_mod_symm(Nat.3.pow(Nat.2), Nat.2, Nat.7)
        Nat.2.congr_mod(Nat.3.pow(Nat.2), Nat.7)
        congr_mod_trans(Nat.2, Nat.3.pow(Nat.2), Nat.1, Nat.7)
        Nat.2.congr_mod(Nat.1, Nat.7)
        false
    }
}

/// `3^3 ≢ 1 (mod 7)`.
theorem not_congr_three_pow_three_mod_seven {
    not Nat.3.pow(Nat.3).congr_mod(Nat.1, Nat.7)
} by {
    congr_three_pow_three_mod_seven
    Nat.3.pow(Nat.3).congr_mod(Nat.6, Nat.7)
    not_congr_six_mod_seven
    if Nat.3.pow(Nat.3).congr_mod(Nat.1, Nat.7) {
        congr_mod_symm(Nat.3.pow(Nat.3), Nat.6, Nat.7)
        Nat.6.congr_mod(Nat.3.pow(Nat.3), Nat.7)
        congr_mod_trans(Nat.6, Nat.3.pow(Nat.3), Nat.1, Nat.7)
        Nat.6.congr_mod(Nat.1, Nat.7)
        false
    }
}

/// `3^4 ≢ 1 (mod 7)`.
theorem not_congr_three_pow_four_mod_seven {
    not Nat.3.pow(Nat.4).congr_mod(Nat.1, Nat.7)
} by {
    congr_three_pow_four_mod_seven
    Nat.3.pow(Nat.4).congr_mod(Nat.4, Nat.7)
    not_congr_four_mod_seven
    if Nat.3.pow(Nat.4).congr_mod(Nat.1, Nat.7) {
        congr_mod_symm(Nat.3.pow(Nat.4), Nat.4, Nat.7)
        Nat.4.congr_mod(Nat.3.pow(Nat.4), Nat.7)
        congr_mod_trans(Nat.4, Nat.3.pow(Nat.4), Nat.1, Nat.7)
        Nat.4.congr_mod(Nat.1, Nat.7)
        false
    }
}

/// `3^5 ≢ 1 (mod 7)`.
theorem not_congr_three_pow_five_mod_seven {
    not Nat.3.pow(Nat.5).congr_mod(Nat.1, Nat.7)
} by {
    congr_three_pow_five_mod_seven
    Nat.3.pow(Nat.5).congr_mod(Nat.5, Nat.7)
    not_congr_five_mod_seven
    if Nat.3.pow(Nat.5).congr_mod(Nat.1, Nat.7) {
        congr_mod_symm(Nat.3.pow(Nat.5), Nat.5, Nat.7)
        Nat.5.congr_mod(Nat.3.pow(Nat.5), Nat.7)
        congr_mod_trans(Nat.5, Nat.3.pow(Nat.5), Nat.1, Nat.7)
        Nat.5.congr_mod(Nat.1, Nat.7)
        false
    }
}

/// The powers of `3` modulo `7`: `3^1 ≡ 3`, `3^2 ≡ 2`, `3^3 ≡ 6`,
/// `3^4 ≡ 4`, `3^5 ≡ 5`, and `3^6 ≡ 1`.
theorem three_powers_mod_seven_table {
    Nat.3.pow(Nat.1).congr_mod(Nat.3, Nat.7) and
        Nat.3.pow(Nat.2).congr_mod(Nat.2, Nat.7) and
        Nat.3.pow(Nat.3).congr_mod(Nat.6, Nat.7) and
        Nat.3.pow(Nat.4).congr_mod(Nat.4, Nat.7) and
        Nat.3.pow(Nat.5).congr_mod(Nat.5, Nat.7) and
        Nat.3.pow(Nat.6).congr_mod(Nat.1, Nat.7)
} by {
    congr_three_pow_one_mod_seven
    Nat.3.pow(Nat.1).congr_mod(Nat.3, Nat.7)
    congr_three_pow_two_mod_seven
    Nat.3.pow(Nat.2).congr_mod(Nat.2, Nat.7)
    congr_three_pow_three_mod_seven
    Nat.3.pow(Nat.3).congr_mod(Nat.6, Nat.7)
    congr_three_pow_four_mod_seven
    Nat.3.pow(Nat.4).congr_mod(Nat.4, Nat.7)
    congr_three_pow_five_mod_seven
    Nat.3.pow(Nat.5).congr_mod(Nat.5, Nat.7)
    congr_three_pow_six_mod_seven
    Nat.3.pow(Nat.6).congr_mod(Nat.1, Nat.7)
    Nat.3.pow(Nat.1).congr_mod(Nat.3, Nat.7) and
        Nat.3.pow(Nat.2).congr_mod(Nat.2, Nat.7) and
        Nat.3.pow(Nat.3).congr_mod(Nat.6, Nat.7) and
        Nat.3.pow(Nat.4).congr_mod(Nat.4, Nat.7) and
        Nat.3.pow(Nat.5).congr_mod(Nat.5, Nat.7) and
        Nat.3.pow(Nat.6).congr_mod(Nat.1, Nat.7)
}

/// `3` has multiplicative order `6` modulo `7`.
theorem order_three_mod_seven {
    multiplicative_order_mod(Nat.3, Nat.7) = Nat.6
} by {
    Nat.7 != Nat.0
    three_coprime_mod_seven
    Nat.3.coprime(Nat.7)
    lt_zero_six
    Nat.0 < Nat.6
    congr_three_pow_six_mod_seven
    Nat.3.pow(Nat.6).congr_mod(Nat.1, Nat.7)
    multiplicative_order_mod_minimal(Nat.3, Nat.7, Nat.6)
    multiplicative_order_mod(Nat.3, Nat.7) <= Nat.6
    multiplicative_order_mod_positive(Nat.3, Nat.7)
    Nat.0 < multiplicative_order_mod(Nat.3, Nat.7)
    if multiplicative_order_mod(Nat.3, Nat.7) = Nat.1 {
        multiplicative_order_mod_is_order(Nat.3, Nat.7)
        is_multiplicative_order_mod(Nat.3, Nat.7, multiplicative_order_mod(Nat.3, Nat.7))
        is_multiplicative_order_mod(Nat.3, Nat.7, Nat.1)
        multiplicative_order_pow_congr_one(Nat.3, Nat.7, Nat.1)
        Nat.3.pow(Nat.1).congr_mod(Nat.1, Nat.7)
        not_congr_three_pow_one_mod_seven
        false
    }
    if multiplicative_order_mod(Nat.3, Nat.7) = Nat.2 {
        multiplicative_order_mod_is_order(Nat.3, Nat.7)
        is_multiplicative_order_mod(Nat.3, Nat.7, multiplicative_order_mod(Nat.3, Nat.7))
        is_multiplicative_order_mod(Nat.3, Nat.7, Nat.2)
        multiplicative_order_pow_congr_one(Nat.3, Nat.7, Nat.2)
        Nat.3.pow(Nat.2).congr_mod(Nat.1, Nat.7)
        not_congr_three_pow_two_mod_seven
        false
    }
    if multiplicative_order_mod(Nat.3, Nat.7) = Nat.3 {
        multiplicative_order_mod_is_order(Nat.3, Nat.7)
        is_multiplicative_order_mod(Nat.3, Nat.7, multiplicative_order_mod(Nat.3, Nat.7))
        is_multiplicative_order_mod(Nat.3, Nat.7, Nat.3)
        multiplicative_order_pow_congr_one(Nat.3, Nat.7, Nat.3)
        Nat.3.pow(Nat.3).congr_mod(Nat.1, Nat.7)
        not_congr_three_pow_three_mod_seven
        false
    }
    if multiplicative_order_mod(Nat.3, Nat.7) = Nat.4 {
        multiplicative_order_mod_is_order(Nat.3, Nat.7)
        is_multiplicative_order_mod(Nat.3, Nat.7, multiplicative_order_mod(Nat.3, Nat.7))
        is_multiplicative_order_mod(Nat.3, Nat.7, Nat.4)
        multiplicative_order_pow_congr_one(Nat.3, Nat.7, Nat.4)
        Nat.3.pow(Nat.4).congr_mod(Nat.1, Nat.7)
        not_congr_three_pow_four_mod_seven
        false
    }
    if multiplicative_order_mod(Nat.3, Nat.7) = Nat.5 {
        multiplicative_order_mod_is_order(Nat.3, Nat.7)
        is_multiplicative_order_mod(Nat.3, Nat.7, multiplicative_order_mod(Nat.3, Nat.7))
        is_multiplicative_order_mod(Nat.3, Nat.7, Nat.5)
        multiplicative_order_pow_congr_one(Nat.3, Nat.7, Nat.5)
        Nat.3.pow(Nat.5).congr_mod(Nat.1, Nat.7)
        not_congr_three_pow_five_mod_seven
        false
    }
    trichotomy(multiplicative_order_mod(Nat.3, Nat.7), Nat.6)
    if multiplicative_order_mod(Nat.3, Nat.7) < Nat.6 {
        lt_suc_right(multiplicative_order_mod(Nat.3, Nat.7), Nat.5)
        if multiplicative_order_mod(Nat.3, Nat.7) = Nat.5 {
            false
        }
        multiplicative_order_mod(Nat.3, Nat.7) < Nat.5
        lt_suc_right(multiplicative_order_mod(Nat.3, Nat.7), Nat.4)
        if multiplicative_order_mod(Nat.3, Nat.7) = Nat.4 {
            false
        }
        multiplicative_order_mod(Nat.3, Nat.7) < Nat.4
        lt_suc_right(multiplicative_order_mod(Nat.3, Nat.7), Nat.3)
        if multiplicative_order_mod(Nat.3, Nat.7) = Nat.3 {
            false
        }
        multiplicative_order_mod(Nat.3, Nat.7) < Nat.3
        lt_suc_right(multiplicative_order_mod(Nat.3, Nat.7), Nat.2)
        if multiplicative_order_mod(Nat.3, Nat.7) = Nat.2 {
            false
        }
        multiplicative_order_mod(Nat.3, Nat.7) < Nat.2
        lt_suc_right(multiplicative_order_mod(Nat.3, Nat.7), Nat.1)
        if multiplicative_order_mod(Nat.3, Nat.7) = Nat.1 {
            false
        }
        multiplicative_order_mod(Nat.3, Nat.7) < Nat.1
        lt_suc_right(multiplicative_order_mod(Nat.3, Nat.7), Nat.0)
        if multiplicative_order_mod(Nat.3, Nat.7) = Nat.0 {
            Nat.0 < multiplicative_order_mod(Nat.3, Nat.7)
            Nat.0 < Nat.0
            lt_not_ref(Nat.0)
            false
        }
        multiplicative_order_mod(Nat.3, Nat.7) < Nat.0
        not_lt_zero(multiplicative_order_mod(Nat.3, Nat.7))
        false
    }
    if Nat.6 < multiplicative_order_mod(Nat.3, Nat.7) {
        lte_imp_not_lt(multiplicative_order_mod(Nat.3, Nat.7), Nat.6)
        not Nat.6 < multiplicative_order_mod(Nat.3, Nat.7)
        false
    }
    multiplicative_order_mod(Nat.3, Nat.7) = Nat.6
}

/// `3` is a primitive root modulo `7`: it is coprime to `7` and its order
/// `6` is the full order `φ(7) = 7 - 1`.
theorem three_is_primitive_root_mod_seven {
    is_primitive_root_mod(Nat.3, Nat.7)
} by {
    seven_is_prime
    Nat.7.is_prime
    three_coprime_mod_seven
    Nat.3.coprime(Nat.7)
    order_three_mod_seven
    multiplicative_order_mod(Nat.3, Nat.7) = Nat.6
    add_one_right(Nat.6)
    Nat.6 + Nat.1 = Nat.7
    add_imp_sub(Nat.6, Nat.1, Nat.7)
    Nat.7 - Nat.1 = Nat.6
    multiplicative_order_mod(Nat.3, Nat.7) = Nat.7 - Nat.1
    Nat.7.is_prime and Nat.3.coprime(Nat.7) and
        multiplicative_order_mod(Nat.3, Nat.7) = Nat.7 - Nat.1
    is_primitive_root_mod(Nat.3, Nat.7) =
        (Nat.7.is_prime and Nat.3.coprime(Nat.7) and
            multiplicative_order_mod(Nat.3, Nat.7) = Nat.7 - Nat.1)
    is_primitive_root_mod(Nat.3, Nat.7)
}

/// The order of `3` modulo `7` divides `7 - 1 = 6` (the prime-order
/// divisibility law, cross-checking the computed order).
theorem order_three_mod_seven_divides_six {
    multiplicative_order_mod(Nat.3, Nat.7).divides(Nat.6)
} by {
    seven_is_prime
    Nat.7.is_prime
    three_coprime_mod_seven
    Nat.3.coprime(Nat.7)
    order_divides_prime_minus_one(Nat.3, Nat.7)
    multiplicative_order_mod(Nat.3, Nat.7).divides(Nat.7 - Nat.1)
    add_one_right(Nat.6)
    Nat.6 + Nat.1 = Nat.7
    add_imp_sub(Nat.6, Nat.1, Nat.7)
    Nat.7 - Nat.1 = Nat.6
    multiplicative_order_mod(Nat.3, Nat.7).divides(Nat.6)
}

// ---------------------------------------------------------------------------
// The period law: a^k ≡ 1 (mod n) iff ord_n(a) | k.
// ---------------------------------------------------------------------------

/// For `a` coprime to `n`, the powers congruent to `1` modulo `n` are exactly
/// those whose exponent is a multiple of the order: `a^k ≡ 1 (mod n)` iff
/// `ord_n(a) | k`.
theorem order_divides_exponent_iff(a: Nat, n: Nat, k: Nat) {
    n != Nat.0 and a.coprime(n) implies
        (multiplicative_order_mod(a, n).divides(k) =
            a.pow(k).congr_mod(Nat.1, n))
} by {
    if n != Nat.0 and a.coprime(n) {
        multiplicative_order_mod_divides_exponent_iff(a, n, k)
        multiplicative_order_mod(a, n).divides(k) = a.pow(k).congr_mod(Nat.1, n)
    }
}

/// `4` divides `8`.
theorem four_divides_eight {
    Nat.4.divides(Nat.8)
} by {
    nat_mul_4_2
    Nat.4 * Nat.2 = Nat.8
    exists(c: Nat) { Nat.4 * c = Nat.8 }
    Nat.4.divides(Nat.8)
}

/// The order of `2` modulo `5` divides `8`: it is `4` and `4 | 8`.
theorem order_two_mod_five_divides_eight {
    multiplicative_order_mod(Nat.2, Nat.5).divides(Nat.8)
} by {
    order_two_mod_five
    multiplicative_order_mod(Nat.2, Nat.5) = Nat.4
    four_divides_eight
    Nat.4.divides(Nat.8)
    multiplicative_order_mod(Nat.2, Nat.5).divides(Nat.8)
}

/// `2^8 ≡ 1 (mod 5)`: since `2^4 ≡ 1 (mod 5)`, squaring gives `2^8 ≡ 1`.
theorem congr_two_pow_eight_mod_five {
    Nat.2.pow(Nat.8).congr_mod(Nat.1, Nat.5)
} by {
    congr_two_pow_four_mod_five
    Nat.2.pow(Nat.4).congr_mod(Nat.1, Nat.5)
    congr_mod_pow(Nat.2.pow(Nat.4), Nat.1, Nat.5, Nat.2)
    Nat.2.pow(Nat.4).pow(Nat.2).congr_mod(Nat.1.pow(Nat.2), Nat.5)
    exp_mul(Nat.2, Nat.4, Nat.2)
    Nat.2.pow(Nat.4 * Nat.2) = Nat.2.pow(Nat.4).pow(Nat.2)
    nat_mul_4_2
    Nat.4 * Nat.2 = Nat.8
    Nat.2.pow(Nat.8) = Nat.2.pow(Nat.4).pow(Nat.2)
    one_exp(Nat.2)
    Nat.1.pow(Nat.2) = Nat.1
    Nat.2.pow(Nat.8).congr_mod(Nat.1, Nat.5)
}

/// `2^8 ≡ 1 (mod 5)` implies `ord_5(2) | 8`.
theorem two_pow_eight_congr_one_imp_order_divides_eight {
    Nat.2.pow(Nat.8).congr_mod(Nat.1, Nat.5)
        implies multiplicative_order_mod(Nat.2, Nat.5).divides(Nat.8)
} by {
    if Nat.2.pow(Nat.8).congr_mod(Nat.1, Nat.5) {
        Nat.5 != Nat.0
        two_coprime_mod_five
        Nat.2.coprime(Nat.5)
        order_divides_exponent_iff(Nat.2, Nat.5, Nat.8)
        multiplicative_order_mod(Nat.2, Nat.5).divides(Nat.8) =
            Nat.2.pow(Nat.8).congr_mod(Nat.1, Nat.5)
        multiplicative_order_mod(Nat.2, Nat.5).divides(Nat.8)
    }
}

/// `ord_5(2) | 8` implies `2^8 ≡ 1 (mod 5)`.
theorem order_divides_eight_imp_two_pow_eight_congr_one {
    multiplicative_order_mod(Nat.2, Nat.5).divides(Nat.8)
        implies Nat.2.pow(Nat.8).congr_mod(Nat.1, Nat.5)
} by {
    if multiplicative_order_mod(Nat.2, Nat.5).divides(Nat.8) {
        Nat.5 != Nat.0
        two_coprime_mod_five
        Nat.2.coprime(Nat.5)
        order_divides_exponent_iff(Nat.2, Nat.5, Nat.8)
        multiplicative_order_mod(Nat.2, Nat.5).divides(Nat.8) =
            Nat.2.pow(Nat.8).congr_mod(Nat.1, Nat.5)
        Nat.2.pow(Nat.8).congr_mod(Nat.1, Nat.5)
    }
}

/// The period of `2` modulo `5`: the order is `4`, the power `2^4` is
/// congruent to `1`, and the exponent `4` is a multiple of the order.
theorem order_two_mod_five_period_four {
    multiplicative_order_mod(Nat.2, Nat.5) = Nat.4 and
        Nat.2.pow(Nat.4).congr_mod(Nat.1, Nat.5) and
        multiplicative_order_mod(Nat.2, Nat.5).divides(Nat.4)
} by {
    order_two_mod_five
    multiplicative_order_mod(Nat.2, Nat.5) = Nat.4
    congr_two_pow_four_mod_five
    Nat.2.pow(Nat.4).congr_mod(Nat.1, Nat.5)
    divides_self(Nat.4)
    Nat.4.divides(Nat.4)
    multiplicative_order_mod(Nat.2, Nat.5).divides(Nat.4)
    multiplicative_order_mod(Nat.2, Nat.5) = Nat.4 and
        Nat.2.pow(Nat.4).congr_mod(Nat.1, Nat.5) and
        multiplicative_order_mod(Nat.2, Nat.5).divides(Nat.4)
}

// ---------------------------------------------------------------------------
// The number of primitive roots modulo p.
//
// The classical count — there are exactly φ(p - 1) primitive roots modulo
// the prime p:
//
//   theorem primitive_root_count_mod_prime(p: Nat) {
//       p.is_prime implies
//           p.range.filter(function(x: Nat) {
//               multiplicative_order_mod(x, p) = p - Nat.1
//           }).length = (p - Nat.1).totient
//   }
//
// — is left as a statement: the proof needs the identity that the exponents
// `k < p - 1` with `gcd(p - 1, k) = (p - 1)/d` number `φ(d)` for every
// divisor `d` of `p - 1`, which the library does not yet have (see the note
// in primitive_root_applications2.ac). The small case `p = 5` is verified
// below: `φ(4) = 2`, and the primitive roots modulo `5` are exactly `2`
// and `3`.
// ---------------------------------------------------------------------------

/// `φ(4) = 2`: Euler's totient of `4` is `2`.
theorem totient_four_is_two {
    Nat.4.totient = Nat.2
} by {
    totient_four_mod_five
    Nat.4.totient = Nat.2
}

/// `φ(5 - 1) = φ(4) = 2`.
theorem totient_of_five_minus_one {
    (Nat.5 - Nat.1).totient = Nat.2
} by {
    four_plus_one
    Nat.4 + Nat.1 = Nat.5
    add_imp_sub(Nat.4, Nat.1, Nat.5)
    Nat.5 - Nat.1 = Nat.4
    totient_four_mod_five
    Nat.4.totient = Nat.2
    (Nat.5 - Nat.1).totient = Nat.2
}

/// `3` is a primitive root modulo `5`.
theorem three_is_primitive_root_mod_five {
    is_primitive_root_mod(Nat.3, Nat.5)
} by {
    five_is_prime
    Nat.5.is_prime
    three_coprime_mod_five
    Nat.3.coprime(Nat.5)
    order_three_mod_five
    multiplicative_order_mod(Nat.3, Nat.5) = Nat.4
    four_plus_one
    Nat.4 + Nat.1 = Nat.5
    add_imp_sub(Nat.4, Nat.1, Nat.5)
    Nat.5 - Nat.1 = Nat.4
    multiplicative_order_mod(Nat.3, Nat.5) = Nat.5 - Nat.1
    Nat.5.is_prime and Nat.3.coprime(Nat.5) and
        multiplicative_order_mod(Nat.3, Nat.5) = Nat.5 - Nat.1
    is_primitive_root_mod(Nat.3, Nat.5) =
        (Nat.5.is_prime and Nat.3.coprime(Nat.5) and
            multiplicative_order_mod(Nat.3, Nat.5) = Nat.5 - Nat.1)
    is_primitive_root_mod(Nat.3, Nat.5)
}

/// `1 != 4`.
theorem one_ne_four {
    Nat.1 != Nat.4
} by {
    lt_ne(Nat.1, Nat.4, Nat.3)
    one_plus_three
    Nat.1 + Nat.3 = Nat.4
    Nat.3 != Nat.0
    Nat.1 != Nat.4
}

/// `2 != 4`.
theorem two_ne_four {
    Nat.2 != Nat.4
} by {
    lt_ne(Nat.2, Nat.4, Nat.2)
    two_plus_two
    Nat.2 + Nat.2 = Nat.4
    Nat.2 != Nat.0
    Nat.2 != Nat.4
}

/// `0` is not a primitive root modulo `5`.
theorem zero_not_primitive_root_mod_five {
    not is_primitive_root_mod(Nat.0, Nat.5)
} by {
    if is_primitive_root_mod(Nat.0, Nat.5) {
        is_primitive_root_mod(Nat.0, Nat.5) =
            (Nat.5.is_prime and Nat.0.coprime(Nat.5) and
                multiplicative_order_mod(Nat.0, Nat.5) = Nat.5 - Nat.1)
        Nat.0.coprime(Nat.5)
        zero_not_coprime_mod_five
        not Nat.0.coprime(Nat.5)
        false
    }
}

/// `1` is not a primitive root modulo `5`.
theorem one_not_primitive_root_mod_five {
    not is_primitive_root_mod(Nat.1, Nat.5)
} by {
    if is_primitive_root_mod(Nat.1, Nat.5) {
        is_primitive_root_mod(Nat.1, Nat.5) =
            (Nat.5.is_prime and Nat.1.coprime(Nat.5) and
                multiplicative_order_mod(Nat.1, Nat.5) = Nat.5 - Nat.1)
        multiplicative_order_mod(Nat.1, Nat.5) = Nat.5 - Nat.1
        four_plus_one
        Nat.4 + Nat.1 = Nat.5
        add_imp_sub(Nat.4, Nat.1, Nat.5)
        Nat.5 - Nat.1 = Nat.4
        multiplicative_order_mod(Nat.1, Nat.5) = Nat.4
        order_one_mod_five
        multiplicative_order_mod(Nat.1, Nat.5) = Nat.1
        Nat.1 = Nat.4
        one_ne_four
        Nat.1 != Nat.4
        false
    }
}

/// `4` is not a primitive root modulo `5`.
theorem four_not_primitive_root_mod_five {
    not is_primitive_root_mod(Nat.4, Nat.5)
} by {
    if is_primitive_root_mod(Nat.4, Nat.5) {
        is_primitive_root_mod(Nat.4, Nat.5) =
            (Nat.5.is_prime and Nat.4.coprime(Nat.5) and
                multiplicative_order_mod(Nat.4, Nat.5) = Nat.5 - Nat.1)
        multiplicative_order_mod(Nat.4, Nat.5) = Nat.5 - Nat.1
        four_plus_one
        Nat.4 + Nat.1 = Nat.5
        add_imp_sub(Nat.4, Nat.1, Nat.5)
        Nat.5 - Nat.1 = Nat.4
        multiplicative_order_mod(Nat.4, Nat.5) = Nat.4
        order_four_mod_five
        multiplicative_order_mod(Nat.4, Nat.5) = Nat.2
        Nat.2 = Nat.4
        two_ne_four
        Nat.2 != Nat.4
        false
    }
}

/// `x < 5` forces `x` into `{0, 1, 2, 3, 4}`.
theorem k_below_five_cases(k: Nat) {
    k < Nat.5 implies (k = Nat.0 or k = Nat.1 or k = Nat.2 or k = Nat.3 or k = Nat.4)
} by {
    if k < Nat.5 {
        lt_suc_right(k, Nat.4)
        k = Nat.4 or k < Nat.4
        if k < Nat.4 {
            lt_suc_right(k, Nat.3)
            k = Nat.3 or k < Nat.3
            if k < Nat.3 {
                lt_suc_right(k, Nat.2)
                k = Nat.2 or k < Nat.2
                if k < Nat.2 {
                    lt_suc_right(k, Nat.1)
                    k = Nat.1 or k < Nat.1
                    if k < Nat.1 {
                        lt_suc_right(k, Nat.0)
                        k = Nat.0 or k < Nat.0
                        if k < Nat.0 {
                            not_lt_zero(k)
                            false
                        } else {
                            k = Nat.0
                            k = Nat.0 or k = Nat.1 or k = Nat.2 or k = Nat.3 or k = Nat.4
                        }
                    } else {
                        k = Nat.1
                        k = Nat.0 or k = Nat.1 or k = Nat.2 or k = Nat.3 or k = Nat.4
                    }
                } else {
                    k = Nat.2
                    k = Nat.0 or k = Nat.1 or k = Nat.2 or k = Nat.3 or k = Nat.4
                }
            } else {
                k = Nat.3
                k = Nat.0 or k = Nat.1 or k = Nat.2 or k = Nat.3 or k = Nat.4
            }
        } else {
            k = Nat.4
            k = Nat.0 or k = Nat.1 or k = Nat.2 or k = Nat.3 or k = Nat.4
        }
    }
}

/// A primitive root below `5` is either `2` or `3`.
theorem primitive_root_mod_five_imp_two_or_three(x: Nat) {
    x < Nat.5 implies (is_primitive_root_mod(x, Nat.5) implies (x = Nat.2 or x = Nat.3))
} by {
    if x < Nat.5 {
        if is_primitive_root_mod(x, Nat.5) {
            k_below_five_cases(x)
            x = Nat.0 or x = Nat.1 or x = Nat.2 or x = Nat.3 or x = Nat.4
            if x = Nat.0 {
                zero_not_primitive_root_mod_five
                not is_primitive_root_mod(Nat.0, Nat.5)
                false
            } else {
                (x = Nat.1 or x = Nat.2 or x = Nat.3 or x = Nat.4)
                if x = Nat.1 {
                    one_not_primitive_root_mod_five
                    not is_primitive_root_mod(Nat.1, Nat.5)
                    false
                } else {
                    (x = Nat.2 or x = Nat.3 or x = Nat.4)
                    if x = Nat.2 {
                        x = Nat.2 or x = Nat.3
                    } else {
                        (x = Nat.3 or x = Nat.4)
                        if x = Nat.3 {
                            x = Nat.2 or x = Nat.3
                        } else {
                            x = Nat.4
                            four_not_primitive_root_mod_five
                            not is_primitive_root_mod(Nat.4, Nat.5)
                            false
                        }
                    }
                }
            }
            x = Nat.2 or x = Nat.3
        }
    }
}

/// Every `x` below `5` equal to `2` or `3` is a primitive root modulo `5`.
theorem two_or_three_imp_primitive_root_mod_five(x: Nat) {
    x < Nat.5 implies ((x = Nat.2 or x = Nat.3) implies is_primitive_root_mod(x, Nat.5))
} by {
    if x < Nat.5 {
        if x = Nat.2 or x = Nat.3 {
            if x = Nat.2 {
                two_is_primitive_root_mod_five
                is_primitive_root_mod(Nat.2, Nat.5)
                is_primitive_root_mod(x, Nat.5)
            } else {
                three_is_primitive_root_mod_five
                is_primitive_root_mod(Nat.3, Nat.5)
                is_primitive_root_mod(x, Nat.5)
            }
            is_primitive_root_mod(x, Nat.5)
        }
    }
}

/// The primitive roots modulo `5` are exactly `2` and `3`.
theorem primitive_root_mod_five_iff_two_or_three(x: Nat) {
    x < Nat.5 implies (is_primitive_root_mod(x, Nat.5) = (x = Nat.2 or x = Nat.3))
} by {
    if x < Nat.5 {
        if is_primitive_root_mod(x, Nat.5) {
            primitive_root_mod_five_imp_two_or_three(x)
            x = Nat.2 or x = Nat.3
            eq_true_intro(is_primitive_root_mod(x, Nat.5))
            is_primitive_root_mod(x, Nat.5) = true
            eq_true_intro(x = Nat.2 or x = Nat.3)
            (x = Nat.2 or x = Nat.3) = true
            is_primitive_root_mod(x, Nat.5) = (x = Nat.2 or x = Nat.3)
        }
        if not is_primitive_root_mod(x, Nat.5) {
            if x = Nat.2 or x = Nat.3 {
                two_or_three_imp_primitive_root_mod_five(x)
                is_primitive_root_mod(x, Nat.5)
                false
            }
            not (x = Nat.2 or x = Nat.3)
            eq_false_intro(is_primitive_root_mod(x, Nat.5))
            is_primitive_root_mod(x, Nat.5) = false
            eq_false_intro(x = Nat.2 or x = Nat.3)
            (x = Nat.2 or x = Nat.3) = false
            is_primitive_root_mod(x, Nat.5) = (x = Nat.2 or x = Nat.3)
        }
        is_primitive_root_mod(x, Nat.5) = (x = Nat.2 or x = Nat.3)
    }
}

/// The number of elements of order `4` modulo `5` — the primitive roots —
/// is `φ(5 - 1) = φ(4) = 2`.
theorem primitive_root_count_mod_five {
    Nat.5.range.filter(order_four_pred_mod_five).length = (Nat.5 - Nat.1).totient
} by {
    elements_of_order_four_count_mod_five
    Nat.5.range.filter(order_four_pred_mod_five).length = Nat.4.totient
    four_plus_one
    Nat.4 + Nat.1 = Nat.5
    add_imp_sub(Nat.4, Nat.1, Nat.5)
    Nat.5 - Nat.1 = Nat.4
    (Nat.5 - Nat.1).totient = Nat.4.totient
    Nat.5.range.filter(order_four_pred_mod_five).length = (Nat.5 - Nat.1).totient
}
