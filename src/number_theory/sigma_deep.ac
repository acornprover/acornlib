/// Deeper results on the divisor-sum function `sigma(n)`.
///
/// This module collects the classical sigma results that sit one layer above
/// the basic divisor-sum machinery of `divisor_sum.ac`:
///
///   (a) `sigma(n) >= n + 1` for every `n > 1`;
///   (b) `sigma(n) = n + 1` exactly when `n` is prime;
///   (c) the prime-power formula `sigma(p^k) = (p^(k+1) - 1) / (p - 1)`;
///   (e) multiplicativity: `sigma(m * n) = sigma(m) * sigma(n)` for
///       positive coprime `m` and `n`.
///
/// Items (a), (b), (c), and (e) are proved in `sigma_multiplicative.ac`
/// (via the prime-power divisor machinery and the Dirichlet-convolution
/// identity `sigma = id * 1`); this file restates them in one place, so that
/// the statements read as a self-contained deepening of the sigma theory.
///
/// The classical characterisation that `sigma(n)` is odd exactly when `n` is
/// a square or twice a square (item (d)) needs the parity theory of the
/// naturals (mod-2 arithmetic), which the library does not yet provide; it is
/// left as a comment at the end of the file, following `divisor_identities.ac`
/// and `divisor_sum_deep.ac`.
from nat import Nat
from number_theory.divisor_sum import nat_sigma
from number_theory.sigma_multiplicative import nat_sigma_ge_self_plus_one,
    nat_sigma_eq_self_plus_one_iff_prime, nat_sigma_prime_pow,
    nat_sigma_mul_coprime
numerals Nat

// ---------------------------------------------------------------------------
// (a) The divisor sum is at least n + 1.
//
// The divisors `1` and `n` both appear in the divisor list of `n > 1` and are
// distinct, so `sigma(n)` collects at least `n + 1`.  (Restates
// `nat_sigma_ge_self_plus_one` from `sigma_multiplicative.ac`, which splits
// off the divisor `n` and uses the proper divisor `1`.)
// ---------------------------------------------------------------------------

/// `sigma(n) >= n + 1` for every `n > 1`.
theorem sigma_ge_self_plus_one(n: Nat) {
    Nat.1 < n implies n + Nat.1 <= nat_sigma(n)
} by {
    nat_sigma_ge_self_plus_one(n)
    n + Nat.1 <= nat_sigma(n)
}

// ---------------------------------------------------------------------------
// (b) `sigma(n) = n + 1` exactly when `n` is prime.
//
// If `n` is prime its only positive divisors are `1` and `n`, so the divisor
// sum is `n + 1`.  Conversely, if `n > 1` is not prime then `n` has a proper
// divisor `k` with `1 < k < n`; the divisor sum is then at least
// `n + 1 + k > n + 1` (`nat_sigma_ge_self_plus_divisor`), contradicting
// `sigma(n) = n + 1`.  (Restates `nat_sigma_eq_self_plus_one_iff_prime` from
// `sigma_multiplicative.ac`, which carries out exactly this divisor argument
// through `no_proper_divisor_imp_prime`.)
// ---------------------------------------------------------------------------

/// `sigma(n) = n + 1` exactly when `n` is prime.
theorem sigma_eq_self_plus_one_iff_prime(n: Nat) {
    (nat_sigma(n) = n + Nat.1) = n.is_prime
} by {
    nat_sigma_eq_self_plus_one_iff_prime(n)
    (nat_sigma(n) = n + Nat.1) = n.is_prime
}

// ---------------------------------------------------------------------------
// (c) The prime-power formula.
//
// For a prime `p` the divisors of `p^k` are exactly the powers
// `p^0, ..., p^k`, so `sigma(p^k)` is the geometric series
// `1 + p + ... + p^k = (p^(k+1) - 1) / (p - 1)`.  (Restates
// `nat_sigma_prime_pow` from `sigma_multiplicative.ac`; the geometric-series
// identity there is `nat_pow_sum_geometric`.)
// ---------------------------------------------------------------------------

/// `sigma(p^k) = (p^(k+1) - 1) / (p - 1)` for a prime `p`.
theorem sigma_prime_pow_formula(p: Nat, k: Nat) {
    p.is_prime implies
        nat_sigma(p.pow(k)) = (p.pow(k + Nat.1) - Nat.1).div(p - Nat.1)
} by {
    if p.is_prime {
        nat_sigma_prime_pow(p, k)
        nat_sigma(p.pow(k)) = (p.pow(k + Nat.1) - Nat.1).div(p - Nat.1)
    }
}

// ---------------------------------------------------------------------------
// (e) Multiplicativity.
//
// `sigma(m * n) = sigma(m) * sigma(n)` whenever `m` and `n` are positive and
// coprime.  (Restates `nat_sigma_mul_coprime` from `sigma_multiplicative.ac`;
// the proof lives in `number_theory.dirichlet` via the identity
// `sigma = id * 1`.)
// ---------------------------------------------------------------------------

/// `sigma` is multiplicative on positive coprime arguments:
/// `sigma(m * n) = sigma(m) * sigma(n)` whenever `gcd(m, n) = 1`.
theorem sigma_mul_coprime(m: Nat, n: Nat) {
    Nat.0 < m and Nat.0 < n and m.coprime(n) implies
        nat_sigma(m * n) = nat_sigma(m) * nat_sigma(n)
} by {
    if Nat.0 < m and Nat.0 < n and m.coprime(n) {
        nat_sigma_mul_coprime(m, n)
        nat_sigma(m * n) = nat_sigma(m) * nat_sigma(n)
    }
}

// ---------------------------------------------------------------------------
// (d) Oddness of sigma.
//
// The classical characterisation: `sigma(n)` is odd if and only if `n` is a
// square or twice a square.  Proving it needs the parity theory of the
// naturals (mod-2 arithmetic) together with the product formula
// `sigma(n) = prod sigma(p^e)`: `sigma(p^e) = 1 + p + ... + p^e` is odd
// exactly when `p = 2` or (`p` odd and `e` even), so `sigma(n)` is odd
// exactly when every odd prime divides `n` to an even power, i.e. `n` is a
// square or twice a square.  The library has no even/odd development for the
// naturals yet, so the statement is left here as a comment rather than a
// theorem (see `divisor_identities.ac`).
//
// define nat_is_odd(n: Nat) -> Bool {
//     exists(k: Nat) { n = Nat.2 * k + Nat.1 }
// }
//
// theorem nat_sigma_odd_iff_square_or_twice_square(n: Nat) {
//     nat_is_odd(nat_sigma(n)) = (is_square(n) or exists(k: Nat) { n = Nat.2 * (k * k) })
// }
// ---------------------------------------------------------------------------
