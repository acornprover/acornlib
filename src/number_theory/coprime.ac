from nat import Nat
from nat import gcd_comm, gcd_one_left, gcd_one_right, gcd_mult_right, gcd_divides,
    gcd_divides_left, gcd_divides_right, divides_gcd, gcd_zero_left, gcd_zero_right
from nat import divides_mul, divides_mod, divides_unmod, divides_trans
from nat import exp_zero
numerals Nat

/// Coprimality is symmetric in its arguments.
theorem coprime_comm(a: Nat, b: Nat) {
    a.coprime(b) implies b.coprime(a)
} by {
    if a.coprime(b) {
        a.gcd(b) = 1
        gcd_comm(a, b)
        b.gcd(a) = 1
    }
}

/// One is coprime with every natural number on the left.
theorem coprime_one_left(a: Nat) {
    Nat.1.coprime(a)
} by {
    gcd_one_left(a)
}

/// One is coprime with every natural number on the right.
theorem coprime_one_right(a: Nat) {
    a.coprime(Nat.1)
} by {
    gcd_one_right(a)
}

/// If zero is coprime to a natural number, that number is one.
theorem coprime_zero_left_imp_one(a: Nat) {
    Nat.0.coprime(a) implies a = Nat.1
} by {
    if Nat.0.coprime(a) {
        gcd_zero_left(a)
        Nat.0.gcd(a) = a
        Nat.0.gcd(a) = Nat.1
        a = Nat.1
    }
}

/// If a natural number is coprime to zero, that number is one.
theorem coprime_zero_right_imp_one(a: Nat) {
    a.coprime(Nat.0) implies a = Nat.1
} by {
    if a.coprime(Nat.0) {
        gcd_zero_right(a)
        a.gcd(Nat.0) = a
        a.gcd(Nat.0) = Nat.1
        a = Nat.1
    }
}

/// Coprimality is preserved under multiplication on the right.
theorem coprime_mul(a: Nat, b: Nat, c: Nat) {
    a.coprime(b) and a.coprime(c) implies a.coprime(b * c)
} by {
    if a.coprime(b) and a.coprime(c) {
        a.gcd(b) = 1
        a.gcd(c) = 1
        let d = a.gcd(b * c)
        gcd_divides_left(a, b * c)
        gcd_divides_right(a, b * c)
        d.divides(a)
        d.divides(b * c)
        d.divides(a * c)
        gcd_mult_right(a, b, c)
        a.gcd(b) * c = (a * c).gcd(b * c)
        Nat.1 * c = (a * c).gcd(b * c)
        c = (a * c).gcd(b * c)
        gcd_divides(d, a * c, b * c)
        d.divides((a * c).gcd(b * c))
        d.divides(c)
        gcd_divides(d, a, c)
        d.divides(a.gcd(c))
        d.divides(Nat.1)
        let k: Nat satisfy { d * k = 1 }
        d = 1
        a.gcd(b * c) = 1
    }
}

/// True when the `m`th power of a number remains coprime to `n`.
define coprime_pow_right_pred(a: Nat, n: Nat) -> (Nat -> Bool) {
    function(m: Nat) { a.coprime(n) implies a.pow(m).coprime(n) }
}

/// The zeroth power of a number is coprime to every natural number.
theorem coprime_pow_right_zero(a: Nat, n: Nat) {
    coprime_pow_right_pred(a, n)(Nat.0)
} by {
    if a.coprime(n) {
        exp_zero(a)
        a.pow(Nat.0) = Nat.1
        coprime_one_left(n)
        a.pow(Nat.0).coprime(n)
    }
}

/// If one power is coprime to `n`, then the next power is coprime to `n`.
theorem coprime_pow_right_step(a: Nat, n: Nat, m: Nat) {
    coprime_pow_right_pred(a, n)(m) implies coprime_pow_right_pred(a, n)(m.suc)
} by {
    if coprime_pow_right_pred(a, n)(m) {
        if a.coprime(n) {
            coprime_pow_right_pred(a, n)(m) =
                (a.coprime(n) implies a.pow(m).coprime(n))
            a.pow(m).coprime(n)
            coprime_comm(a, n)
            n.coprime(a)
            coprime_comm(a.pow(m), n)
            n.coprime(a.pow(m))
            coprime_mul(n, a, a.pow(m))
            n.coprime(a * a.pow(m))
            coprime_comm(n, a * a.pow(m))
            (a * a.pow(m)).coprime(n)
            a.pow(m.suc) = a * a.pow(m)
            a.pow(m.suc).coprime(n)
        }
        coprime_pow_right_pred(a, n)(m.suc) =
            (a.coprime(n) implies a.pow(m.suc).coprime(n))
        coprime_pow_right_pred(a, n)(m.suc)
    }
}

/// Every power of a number coprime to `n` remains coprime to `n`.
theorem coprime_pow_right(a: Nat, n: Nat, m: Nat) {
    a.coprime(n) implies a.pow(m).coprime(n)
} by {
    let f: Nat -> Bool = function(x: Nat) {
        a.coprime(n) implies a.pow(x).coprime(n)
    }
    forall(x: Nat) {
        coprime_pow_right_pred(a, n)(x) = f(x)
        f(x) = coprime_pow_right_pred(a, n)(x)
    }
    coprime_pow_right_zero(a, n)
    coprime_pow_right_pred(a, n)(Nat.0)
    f(Nat.0)
    forall(x: Nat) {
        if f(x) {
            coprime_pow_right_pred(a, n)(x)
            coprime_pow_right_step(a, n, x)
            coprime_pow_right_pred(a, n)(x.suc)
            f(x.suc)
        }
    }
    Nat.induction(f)
    f(m)
}

/// Every power of a number coprime to `a` remains coprime on the right.
theorem coprime_pow_left(a: Nat, n: Nat, m: Nat) {
    a.coprime(n) implies a.coprime(n.pow(m))
} by {
    if a.coprime(n) {
        coprime_comm(a, n)
        n.coprime(a)
        coprime_pow_right(n, a, m)
        n.pow(m).coprime(a)
        coprime_comm(n.pow(m), a)
        a.coprime(n.pow(m))
    }
}

/// Powers of coprime numbers are coprime.
theorem coprime_pow_pow(a: Nat, b: Nat, m: Nat, n: Nat) {
    a.coprime(b) implies a.pow(m).coprime(b.pow(n))
} by {
    if a.coprime(b) {
        coprime_pow_right(a, b, m)
        a.pow(m).coprime(b)
        coprime_comm(a.pow(m), b)
        b.coprime(a.pow(m))
        coprime_pow_right(b, a.pow(m), n)
        b.pow(n).coprime(a.pow(m))
        coprime_comm(b.pow(n), a.pow(m))
        a.pow(m).coprime(b.pow(n))
    }
}

/// The square of a number coprime to `n` remains coprime to `n`.
theorem coprime_square_right(a: Nat, n: Nat) {
    a.coprime(n) implies a.pow(Nat.2).coprime(n)
} by {
    if a.coprime(n) {
        coprime_pow_right(a, n, Nat.2)
        a.pow(Nat.2).coprime(n)
    }
}

/// A number coprime to `n` remains coprime to the square of `n`.
theorem coprime_square_left(a: Nat, n: Nat) {
    a.coprime(n) implies a.coprime(n.pow(Nat.2))
} by {
    if a.coprime(n) {
        coprime_pow_left(a, n, Nat.2)
        a.coprime(n.pow(Nat.2))
    }
}

/// Squares of coprime numbers are coprime.
theorem coprime_square_square(a: Nat, b: Nat) {
    a.coprime(b) implies a.pow(Nat.2).coprime(b.pow(Nat.2))
} by {
    if a.coprime(b) {
        coprime_pow_pow(a, b, Nat.2, Nat.2)
        a.pow(Nat.2).coprime(b.pow(Nat.2))
    }
}

/// Helper: any divisor of 1 in Nat equals 1.
theorem nat_divides_one_imp_one(d: Nat) {
    d.divides(Nat.1) implies d = Nat.1
} by {
    if d.divides(Nat.1) {
        let k: Nat satisfy { d * k = Nat.1 }
    }
}

/// Forward half: if `a` is coprime to `b * c`, it is coprime to `b`.
theorem coprime_mul_imp_left(a: Nat, b: Nat, c: Nat) {
    a.coprime(b * c) implies a.coprime(b)
} by {
    if a.coprime(b * c) {
        let dab: Nat = a.gcd(b)
        gcd_divides_left(a, b)
        dab.divides(a)
        gcd_divides_right(a, b)
        dab.divides(b)
        divides_mul(b, c, dab)
        dab.divides(b * c)
        divides_gcd(dab, a, b * c)
        dab.divides(a.gcd(b * c))
        a.gcd(b * c) = Nat.1
        dab.divides(Nat.1)
        nat_divides_one_imp_one(dab)
        dab = Nat.1
        a.gcd(b) = Nat.1
    }
}

/// Forward half: if `a` is coprime to `b * c`, it is coprime to `c`.
theorem coprime_mul_imp_right(a: Nat, b: Nat, c: Nat) {
    a.coprime(b * c) implies a.coprime(c)
} by {
    if a.coprime(b * c) {
        b * c = c * b
        a.coprime(c * b)
        coprime_mul_imp_left(a, c, b)
    }
}

/// Coprimality with the product of `b` and `c` decomposes into coprimality
/// with each factor.
theorem coprime_mul_iff(a: Nat, b: Nat, c: Nat) {
    a.coprime(b * c) implies (a.coprime(b) and a.coprime(c))
} by {
    if a.coprime(b * c) {
        coprime_mul_imp_left(a, b, c)
        a.coprime(b)
        coprime_mul_imp_right(a, b, c)
        a.coprime(c)
    }
}

/// Divisors of coprime natural numbers are coprime.
theorem coprime_of_divisors(a: Nat, b: Nat, d: Nat, e: Nat) {
    a.coprime(b) and d.divides(a) and e.divides(b) implies d.coprime(e)
} by {
    if a.coprime(b) and d.divides(a) and e.divides(b) {
        let g: Nat = d.gcd(e)
        gcd_divides_left(d, e)
        g.divides(d)
        divides_trans(g, d, a)
        g.divides(a)
        gcd_divides_right(d, e)
        g.divides(e)
        divides_trans(g, e, b)
        g.divides(b)
        divides_gcd(g, a, b)
        g.divides(a.gcd(b))
        a.gcd(b) = Nat.1
        g.divides(Nat.1)
        nat_divides_one_imp_one(g)
        g = Nat.1
        d.coprime(e)
    }
}

/// Euclid's lemma: if a is coprime to b and divides b * c, then a divides c.
theorem coprime_divides_of_divides_mul(a: Nat, b: Nat, c: Nat) {
    a.coprime(b) and a.divides(b * c) implies a.divides(c)
} by {
    if a.coprime(b) and a.divides(b * c) {
        a.gcd(b) = 1
        gcd_mult_right(a, b, c)
        a.gcd(b) * c = (a * c).gcd(b * c)
        Nat.1 * c = (a * c).gcd(b * c)
        c = (a * c).gcd(b * c)
        a.divides(a * c)
        a.divides(b * c)
        gcd_divides(a, a * c, b * c)
        a.divides((a * c).gcd(b * c))
        a.divides(c)
    }
}

/// Forward divisibility: `gcd(k, n)` divides `gcd(k.mod(n), n)`.
theorem gcd_divides_gcd_mod(k: Nat, n: Nat) {
    k.gcd(n).divides((k.mod(n)).gcd(n))
} by {
    let d: Nat = k.gcd(n)
    gcd_divides_left(k, n)
    d.divides(k)
    gcd_divides_right(k, n)
    d.divides(n)
    divides_mod(k, n, d)
    d.divides(k.mod(n))
    divides_gcd(d, k.mod(n), n)
}

/// Backward divisibility: `gcd(k.mod(n), n)` divides `gcd(k, n)`.
theorem gcd_mod_divides_gcd(k: Nat, n: Nat) {
    (k.mod(n)).gcd(n).divides(k.gcd(n))
} by {
    let d: Nat = (k.mod(n)).gcd(n)
    gcd_divides_left(k.mod(n), n)
    d.divides(k.mod(n))
    gcd_divides_right(k.mod(n), n)
    d.divides(n)
    divides_unmod(d, k, n)
    d.divides(k)
    divides_gcd(d, k, n)
}

/// Reducing a number modulo `n` preserves its coprimality with `n`.
theorem coprime_mod_imp(k: Nat, n: Nat) {
    k.coprime(n) implies (k.mod(n)).coprime(n)
} by {
    if k.coprime(n) {
        k.gcd(n) = Nat.1
        // (k.mod(n)).gcd(n) divides k.gcd(n) = 1, so equals 1.
        gcd_mod_divides_gcd(k, n)
        (k.mod(n)).gcd(n).divides(k.gcd(n))
        (k.mod(n)).gcd(n).divides(Nat.1)
        nat_divides_one_imp_one((k.mod(n)).gcd(n))
        (k.mod(n)).gcd(n) = Nat.1
    }
}

/// Reducing modulo `n` does not introduce coprimality: if the reduction is
/// coprime to `n`, the original was already.
theorem coprime_unmod_imp(k: Nat, n: Nat) {
    (k.mod(n)).coprime(n) implies k.coprime(n)
} by {
    if (k.mod(n)).coprime(n) {
        (k.mod(n)).gcd(n) = Nat.1
        // k.gcd(n) divides (k.mod(n)).gcd(n) = 1, so equals 1.
        gcd_divides_gcd_mod(k, n)
        k.gcd(n).divides((k.mod(n)).gcd(n))
        k.gcd(n).divides(Nat.1)
        nat_divides_one_imp_one(k.gcd(n))
        k.gcd(n) = Nat.1
    }
}

/// Coprimality is invariant under modular reduction.
theorem coprime_mod_iff(k: Nat, n: Nat) {
    k.coprime(n) = (k.mod(n)).coprime(n)
} by {
    if k.coprime(n) {
        coprime_mod_imp(k, n)
    }
    if (k.mod(n)).coprime(n) {
        coprime_unmod_imp(k, n)
    }
}
