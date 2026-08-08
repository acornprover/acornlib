from int import Int, sub_nat, add_sub_nat, sub_nat_zero_right, sub_nat_zero_left, mul_from_nat
from nat import Nat, suc_sub_one, zero_or_suc, add_one_right, lt_cancel_suc, add_zero_right, add_zero_left, add_imp_sub, add_imp_sub_left, mul_cancel_left, alt_suc_ne_zero, mul_assoc, lt_suc, lte_and_lt, alt_induction, lte_imp_not_lt, lt_or_lte, lt_imp_lte_suc, lte_antisymm, lte_suc_suc
from rat import Rat, reduce, reduce_self, reduce_idempotent, neg_num, neg_denom, from_int_num, from_int_denom,
    cross_equals, cross_eq_imp_eq, mul_reduced, add_reduced_same_denom, add_reduced, mul_fractions,
    mul_cancels_div, mul_comm, mul_reduced_int_right, reduce_cancels_left, from_int_zero, mul_int_eq_int_mul, int_from_nat_injective, nat_lt_imp_rat_lt, zero_lt_imp_pos, pos_ne_zero
from combinatorics import binom, binom_n_minus_one, choose_one, choose_symm_sub_form, pascal_suc_unbounded, binom_two_double
from list import List, partial, partial_one, partial_split_last, partial_pointwise_eq, add_length

numerals Nat
numerals Rat

/// The k-th summand in the recurrence for the Bernoulli number B_{m+1},
/// computed from the option holding the k-th previously computed value.
/// A missing entry contributes zero.
define bernoulli_summand_of_option(m: Nat, k: Nat, entry: Option[Rat]) -> Rat {
    match entry {
        Option.none {
            Rat.0
        }
        Option.some(b) {
            Rat.from_nat(m.suc.suc.binom(k)) * b
        }
    }
}

/// The k-th summand in the recurrence for the Bernoulli number B_{m+1}:
/// binom(m+2, k) times the k-th previously computed Bernoulli number.
/// The previously computed values are read from a list, and any index outside
/// the list contributes zero.
define bernoulli_summand(m: Nat, prev: List[Rat], k: Nat) -> Rat {
    bernoulli_summand_of_option(m, k, prev.get_idx(k))
}

/// The Bernoulli number B_{m+1} computed from the list [B_0, ..., B_m] of
/// previously computed values, using the recurrence
/// B_{m+1} = -1/(m+2) * sum_{k=0}^{m} binom(m+2, k) B_k.
define bernoulli_next(m: Nat, prev: List[Rat]) -> Rat {
    -Rat.1 / Rat.from_nat(m.suc.suc) * partial(bernoulli_summand(m, prev), m.suc)
}

/// The list [B_0, ..., B_n] of the first n + 1 Bernoulli numbers.
define bernoulli_values(n: Nat) -> List[Rat] {
    match n {
        Nat.zero {
            List.singleton(Rat.1)
        }
        Nat.suc(m) {
            let prev = bernoulli_values(m)
            prev + List.singleton(bernoulli_next(m, prev))
        }
    }
}

/// The n-th Bernoulli number, defined by B_0 = 1 and the recurrence
/// sum_{k=0}^{n} binom(n+1, k) B_k = 0 for n >= 1.
define bernoulli(n: Nat) -> Rat {
    match n {
        Nat.zero {
            Rat.1
        }
        Nat.suc(m) {
            bernoulli_next(m, bernoulli_values(m))
        }
    }
}

/// The k-th term in the Bernoulli recurrence at index n:
/// binom(n+1, k) * B_k.
define bernoulli_recurrence_term(n: Nat, k: Nat) -> Rat {
    Rat.from_nat(n.suc.binom(k)) * bernoulli(k)
}

/// The negation of a rational built from an integer is the rational of the negated integer.
theorem bernoulli_neg_from_int(n: Int) {
    -Rat.from_int(n) = Rat.from_int(-n)
} by {
    (-Rat.from_int(n)).num = -(Rat.from_int(n)).num
    Rat.from_int(n).num = n
    (-Rat.from_int(n)).num = -n
    (-Rat.from_int(n)).denom = Rat.from_int(n).denom
    Rat.from_int(n).denom = Int.1
    Rat.from_int(-n).num = -n
    Rat.from_int(-n).denom = Int.1
    cross_equals((-Rat.from_int(n)).num, (-Rat.from_int(n)).denom, Rat.from_int(-n).num, Rat.from_int(-n).denom)
    cross_eq_imp_eq(-Rat.from_int(n), Rat.from_int(-n))
}

/// Dividing a rational by an integer divides its numerator in reduced form.
theorem bernoulli_div_from_int(r: Rat, n: Int) {
    r / Rat.from_int(n) = reduce(r.num, r.denom * n)
} by {
    r / Rat.from_int(n) = r * Rat.from_int(n).inverse
    Rat.from_int(n).inverse = reduce(Int.1, n)
    r * reduce(Int.1, n) = reduce(r.num, r.denom) * reduce(Int.1, n)
    mul_reduced(r.num, r.denom, Int.1, n)
    reduce(r.num, r.denom) * reduce(Int.1, n) = reduce(r.num * Int.1, r.denom * n)
    reduce_idempotent(r)
    r * Rat.from_int(n).inverse = reduce(r.num * Int.1, r.denom * n)
    r.num * Int.1 = r.num
    r / Rat.from_int(n) = reduce(r.num, r.denom * n)
}

/// The product of an integer with the reciprocal of another integer is a reduced fraction.
theorem bernoulli_int_over_reduce(a: Int, b: Int) {
    Rat.from_int(a) * reduce(Int.1, b) = reduce(a, b)
} by {
    Rat.from_int(a) = reduce(a, Int.1)
    mul_reduced(a, Int.1, Int.1, b)
    reduce(a, Int.1) * reduce(Int.1, b) = reduce(a * Int.1, Int.1 * b)
    a * Int.1 = a
    Int.1 * b = b
    Rat.from_int(a) * reduce(Int.1, b) = reduce(a, b)
}

/// The zeroth Bernoulli number is one.
theorem bernoulli_zero {
    bernoulli(Nat.0) = Rat.1
} by {
    bernoulli(Nat.0) = Rat.1
}

/// The first Bernoulli number is minus one half.
theorem bernoulli_one {
    bernoulli(Nat.1) = -Rat.1 / Rat.2
} by {
    bernoulli_values(Nat.0) = List.singleton(Rat.1)
    bernoulli(Nat.1) = bernoulli_next(Nat.0, bernoulli_values(Nat.0))
    bernoulli(Nat.1) = bernoulli_next(Nat.0, List.singleton(Rat.1))
    List.singleton(Rat.1) = List.cons(Rat.1, List.nil[Rat])
    not Nat.0 > Nat.0
    List.cons(Rat.1, List.nil[Rat]).get_idx(Nat.0) = Option.some(Rat.1)
    List.singleton(Rat.1).get_idx(Nat.0) = Option.some(Rat.1)
    bernoulli_next(Nat.0, List.singleton(Rat.1)) =
        -Rat.1 / Rat.from_nat(Nat.2) *
        partial(bernoulli_summand(Nat.0, List.singleton(Rat.1)), Nat.1)
    Rat.from_nat(Nat.2) = Rat.2
    partial(bernoulli_summand(Nat.0, List.singleton(Rat.1)), Nat.1) =
        bernoulli_summand(Nat.0, List.singleton(Rat.1), Nat.0)
    bernoulli_summand(Nat.0, List.singleton(Rat.1), Nat.0) =
        bernoulli_summand_of_option(Nat.0, Nat.0, List.singleton(Rat.1).get_idx(Nat.0))
    bernoulli_summand_of_option(Nat.0, Nat.0, Option.some(Rat.1)) =
        Rat.from_nat(Nat.2.binom(Nat.0)) * Rat.1
    Nat.2.binom(Nat.0) = Nat.1
    Rat.from_nat(Nat.1) * Rat.1 = Rat.1
    bernoulli(Nat.1) = -Rat.1 / Rat.2
}

/// The zero-th entry of a cons list is its head.
theorem bernoulli_cons_get_idx_zero[T](head: T, tail: List[T]) {
    List.cons(head, tail).get_idx(Nat.0) = Option.some(head)
} by {
    not Nat.0 > Nat.0
    List.cons(head, tail).get_idx(Nat.0) = Option.some(head)
}

/// The successor index of a cons list is the index of its tail.
theorem bernoulli_cons_get_idx_suc[T](head: T, tail: List[T], i: Nat) {
    List.cons(head, tail).get_idx(i.suc) = tail.get_idx(i)
} by {
    i.suc > Nat.0
    List.cons(head, tail).get_idx(i.suc) = tail.get_idx(i.suc - Nat.1)
    suc_sub_one(i)
    List.cons(head, tail).get_idx(i.suc) = tail.get_idx(i)
}

/// The last entry of a list with a singleton appended is the appended element.
theorem bernoulli_append_get_idx_last[T](xs: List[T], x: T) {
    (xs + List.singleton(x)).get_idx(xs.length) = Option.some(x)
} by {
    define f(ys: List[T]) -> Bool {
        (ys + List.singleton(x)).get_idx(ys.length) = Option.some(x)
    }
    List.singleton(x) = List.cons(x, List.nil[T])
    List.nil[T] + List.singleton(x) = List.singleton(x)
    List.nil[T].length = Nat.0
    not Nat.0 > Nat.0
    List.cons(x, List.nil[T]).get_idx(Nat.0) = Option.some(x)
    List.singleton(x).get_idx(Nat.0) = Option.some(x)
    f(List.nil[T])
    forall(head: T, tail: List[T]) {
        if f(tail) {
            let l = List.cons(head, tail)
            let sa = List.singleton(x)
            l.length = tail.length.suc
            l + sa = List.cons(head, tail + sa)
            not Nat.0 > Nat.0
            tail.length.suc > Nat.0
            List.cons(head, tail + sa).get_idx(tail.length.suc) = (tail + sa).get_idx(tail.length.suc - Nat.1)
            suc_sub_one(tail.length)
            List.cons(head, tail + sa).get_idx(tail.length.suc) = (tail + sa).get_idx(tail.length)
            (l + sa).get_idx(l.length) = (tail + sa).get_idx(tail.length)
            suc_sub_one(tail.length)
            (l + sa).get_idx(l.length) = (tail + sa).get_idx(l.length - Nat.1)
            (tail + sa).get_idx(tail.length) = Option.some(x)
            (l + sa).get_idx(l.length) = Option.some(x)
            f(List.cons(head, tail))
        }
    }
    forall(ys: List[T]) {
        f(ys)
    }
    f(xs)
}

/// Entries of an appended list before the append point match the original list,
/// or the index is past the append point.
theorem bernoulli_append_get_idx_prefix_disj[T](xs: List[T], x: T, kk: Nat) {
    (xs + List.singleton(x)).get_idx(kk) = xs.get_idx(kk) or xs.length <= kk
} by {
    define f(ys: List[T]) -> Bool {
        forall(k: Nat) {
            (ys + List.singleton(x)).get_idx(k) = ys.get_idx(k) or ys.length <= k
        }
    }
    forall(k: Nat) {
        List.nil[T] + List.singleton(x) = List.singleton(x)
        (List.nil[T] + List.singleton(x)).get_idx(k) = List.singleton(x).get_idx(k)
        List.nil[T].length = Nat.0
        Nat.0 <= k
        List.nil[T].length <= k
        (List.nil[T] + List.singleton(x)).get_idx(k) = List.nil[T].get_idx(k) or List.nil[T].length <= k
    }
    f(List.nil[T])
    forall(head: T, tail: List[T]) {
        if f(tail) {
            forall(k: Nat) {
                if k = Nat.0 {
                    not Nat.0 > Nat.0
                    (List.cons(head, tail) + List.singleton(x)).get_idx(Nat.0) = Option.some(head)
                    List.cons(head, tail).get_idx(Nat.0) = Option.some(head)
                    (List.cons(head, tail) + List.singleton(x)).get_idx(k) =
                        List.cons(head, tail).get_idx(k)
                    (List.cons(head, tail) + List.singleton(x)).get_idx(k) =
                        List.cons(head, tail).get_idx(k) or List.cons(head, tail).length <= k
                } else {
                    let i: Nat satisfy {
                        k = i.suc
                    }
                    List.cons(head, tail).length = tail.length.suc
                    if i < tail.length {
                        if tail.length <= i {
                            lte_imp_not_lt(tail.length, i)
                            not i < tail.length
                            false
                        }
                        not tail.length <= i
                        (tail + List.singleton(x)).get_idx(i) = tail.get_idx(i)
                        i.suc > Nat.0
                        List.cons(head, tail + List.singleton(x)).get_idx(i.suc) =
                            (tail + List.singleton(x)).get_idx(i.suc - Nat.1)
                        suc_sub_one(i)
                        List.cons(head, tail + List.singleton(x)).get_idx(i.suc) =
                            (tail + List.singleton(x)).get_idx(i)
                        List.cons(head, tail) + List.singleton(x) =
                            List.cons(head, tail + List.singleton(x))
                        (List.cons(head, tail) + List.singleton(x)).get_idx(i.suc) =
                            (tail + List.singleton(x)).get_idx(i)
                        i.suc > Nat.0
                        List.cons(head, tail).get_idx(i.suc) = tail.get_idx(i.suc - Nat.1)
                        suc_sub_one(i)
                        List.cons(head, tail).get_idx(i.suc) = tail.get_idx(i)
                        (List.cons(head, tail) + List.singleton(x)).get_idx(k) =
                            List.cons(head, tail).get_idx(k)
                        (List.cons(head, tail) + List.singleton(x)).get_idx(k) =
                            List.cons(head, tail).get_idx(k) or List.cons(head, tail).length <= k
                    } else {
                        lt_or_lte(i, tail.length)
                        tail.length <= i
                        lte_suc_suc(tail.length, i)
                        tail.length.suc <= i.suc
                        List.cons(head, tail).length <= i.suc
                        (List.cons(head, tail) + List.singleton(x)).get_idx(k) =
                            List.cons(head, tail).get_idx(k) or List.cons(head, tail).length <= k
                    }
                }
            }
            f(List.cons(head, tail))
        }
    }
    forall(ys: List[T]) {
        f(ys)
    }
    f(xs)
    forall(k: Nat) {
        (xs + List.singleton(x)).get_idx(k) = xs.get_idx(k) or xs.length <= k
    }
    (xs + List.singleton(x)).get_idx(kk) = xs.get_idx(kk) or xs.length <= kk
}

/// Entries of an appended list before the append point match the original list.
theorem bernoulli_append_get_idx_prefix[T](xs: List[T], x: T, k: Nat) {
    k < xs.length implies (xs + List.singleton(x)).get_idx(k) = xs.get_idx(k)
} by {
    bernoulli_append_get_idx_prefix_disj(xs, x, k)
    if k < xs.length {
        if xs.length <= k {
            lte_imp_not_lt(xs.length, k)
            not k < xs.length
            false
        }
        not xs.length <= k
        (xs + List.singleton(x)).get_idx(k) = xs.get_idx(k)
        k < xs.length implies (xs + List.singleton(x)).get_idx(k) = xs.get_idx(k)
    }
}

/// The length of the list [B_0, ..., B_n] is n + 1.
theorem bernoulli_values_length(n: Nat) {
    bernoulli_values(n).length = n.suc
} by {
    define p(m: Nat) -> Bool {
        bernoulli_values(m).length = m.suc
    }
    bernoulli_values(Nat.0) = List.singleton(Rat.1)
    List.singleton(Rat.1) = List.cons(Rat.1, List.nil[Rat])
    List.nil[Rat].length = Nat.0
    List.cons(Rat.1, List.nil[Rat]).length = Nat.1
    List.singleton(Rat.1).length = Nat.1
    p(Nat.0)
    forall(m: Nat) {
        if p(m) {
            bernoulli_values(m.suc) =
                bernoulli_values(m) + List.singleton(bernoulli_next(m, bernoulli_values(m)))
            add_length(bernoulli_values(m), List.singleton(bernoulli_next(m, bernoulli_values(m))))
            bernoulli_values(m.suc).length =
                bernoulli_values(m).length +
                List.singleton(bernoulli_next(m, bernoulli_values(m))).length
            List.singleton(bernoulli_next(m, bernoulli_values(m))).length = Nat.1
            bernoulli_values(m.suc).length = bernoulli_values(m).length + Nat.1
            add_one_right(bernoulli_values(m).length)
            bernoulli_values(m).length + Nat.1 = bernoulli_values(m).length.suc
            bernoulli_values(m.suc).length = bernoulli_values(m).length.suc
            bernoulli_values(m).length = m.suc
            bernoulli_values(m).length.suc = m.suc.suc
            bernoulli_values(m.suc).length = m.suc.suc
            p(m.suc)
        }
    }
    forall(m: Nat) {
        p(m) implies p(m.suc)
    }
    p(Nat.0) and forall(m: Nat) { p(m) implies p(m.suc) }
    alt_induction(p)
    forall(m: Nat) {
        p(m)
    }
    p(n)
}

/// The summand computed from a present entry is binom(m+2, k) times that entry.
theorem bernoulli_summand_of_option_some(m: Nat, k: Nat, b: Rat) {
    bernoulli_summand_of_option(m, k, Option.some(b)) =
        Rat.from_nat(m.suc.suc.binom(k)) * b
} by {
    bernoulli_summand_of_option(m, k, Option.some(b)) =
        Rat.from_nat(m.suc.suc.binom(k)) * b
}

/// Every entry of the list [B_0, ..., B_n] is the corresponding Bernoulli number.
theorem bernoulli_values_consistent(n: Nat, kk: Nat) {
    kk <= n implies bernoulli_values(n).get_idx(kk) = Option.some(bernoulli(kk))
} by {
    define p(m: Nat) -> Bool {
        forall(k: Nat) {
            k <= m implies bernoulli_values(m).get_idx(k) = Option.some(bernoulli(k))
        }
    }
    forall(k: Nat) {
        if k <= Nat.0 {
            k = Nat.0
            bernoulli_values(Nat.0) = List.singleton(Rat.1)
            List.singleton(Rat.1) = List.cons(Rat.1, List.nil[Rat])
            not Nat.0 > Nat.0
            List.cons(Rat.1, List.nil[Rat]).get_idx(Nat.0) = Option.some(Rat.1)
            bernoulli_values(Nat.0).get_idx(Nat.0) = Option.some(Rat.1)
            bernoulli(Nat.0) = Rat.1
            Option.some(bernoulli(Nat.0)) = Option.some(Rat.1)
            bernoulli_values(Nat.0).get_idx(k) = Option.some(bernoulli(k))
        }
    }
    p(Nat.0)
    forall(m: Nat) {
        if p(m) {
            forall(k: Nat) {
                if k <= m.suc {
                    if k <= m {
                        bernoulli_values(m).length = m.suc
                        k < bernoulli_values(m).length
                        bernoulli_append_get_idx_prefix(bernoulli_values(m),
                            bernoulli_next(m, bernoulli_values(m)), k)
                        bernoulli_values(m.suc) =
                            bernoulli_values(m) + List.singleton(bernoulli_next(m, bernoulli_values(m)))
                        bernoulli_values(m.suc).get_idx(k) = bernoulli_values(m).get_idx(k)
                        bernoulli_values(m).get_idx(k) = Option.some(bernoulli(k))
                        bernoulli_values(m.suc).get_idx(k) = Option.some(bernoulli(k))
                    } else {
                        k <= m.suc
                        not k <= m
                        lt_or_lte(m, k)
                        m < k or k <= m
                        m < k
                        lt_imp_lte_suc(m, k)
                        m.suc <= k
                        k <= m.suc and m.suc <= k
                        lte_antisymm(k, m.suc)
                        k = m.suc
                        bernoulli_values(m).length = m.suc
                        bernoulli_append_get_idx_last(bernoulli_values(m),
                            bernoulli_next(m, bernoulli_values(m)))
                        bernoulli_values(m.suc) =
                            bernoulli_values(m) + List.singleton(bernoulli_next(m, bernoulli_values(m)))
                        bernoulli_values(m.suc).get_idx(m.suc) =
                            Option.some(bernoulli_next(m, bernoulli_values(m)))
                        bernoulli(m.suc) = bernoulli_next(m, bernoulli_values(m))
                        bernoulli_values(m.suc).get_idx(k) = Option.some(bernoulli(k))
                    }
                }
            }
            p(m.suc)
        }
    }
    p(n)
    kk <= n implies bernoulli_values(n).get_idx(kk) = Option.some(bernoulli(kk))
}

/// The k-th summand agrees with the k-th recurrence term while k is in range.
theorem bernoulli_summand_eq_term(m: Nat, k: Nat) {
    k <= m implies bernoulli_summand(m, bernoulli_values(m), k) =
        Rat.from_nat(m.suc.suc.binom(k)) * bernoulli(k)
} by {
    if k <= m {
        bernoulli_summand(m, bernoulli_values(m), k) =
            bernoulli_summand_of_option(m, k, bernoulli_values(m).get_idx(k))
        bernoulli_values(m).get_idx(k) = Option.some(bernoulli(k))
        bernoulli_summand_of_option(m, k, bernoulli_values(m).get_idx(k)) =
            bernoulli_summand_of_option(m, k, Option.some(bernoulli(k)))
        bernoulli_summand_of_option(m, k, Option.some(bernoulli(k))) =
            Rat.from_nat(m.suc.suc.binom(k)) * bernoulli(k)
        bernoulli_summand(m, bernoulli_values(m), k) =
            Rat.from_nat(m.suc.suc.binom(k)) * bernoulli(k)
    }
}

/// The second Bernoulli number is one sixth.
theorem bernoulli_two {
    bernoulli(Nat.2) = Rat.1 / Rat.6
} by {

    bernoulli(Nat.2) = bernoulli_next(Nat.1, bernoulli_values(Nat.1))
    bernoulli_next(Nat.1, bernoulli_values(Nat.1)) =
        -Rat.1 / Rat.from_nat(Nat.3) *
        partial(bernoulli_summand(Nat.1, bernoulli_values(Nat.1)), Nat.2)
    Rat.from_nat(Nat.3) = Rat.3
    partial(bernoulli_summand(Nat.1, bernoulli_values(Nat.1)), Nat.2) =
        partial(bernoulli_summand(Nat.1, bernoulli_values(Nat.1)), Nat.1) +
        bernoulli_summand(Nat.1, bernoulli_values(Nat.1), Nat.1)
    partial(bernoulli_summand(Nat.1, bernoulli_values(Nat.1)), Nat.1) =
        bernoulli_summand(Nat.1, bernoulli_values(Nat.1), Nat.0)
    Nat.0 <= Nat.1
    bernoulli_summand(Nat.1, bernoulli_values(Nat.1), Nat.0) =
        Rat.from_nat(Nat.3.binom(Nat.0)) * bernoulli(Nat.0)
    Nat.3.binom(Nat.0) = Nat.1
    Rat.from_nat(Nat.1) * Rat.1 = Rat.1
    bernoulli_summand(Nat.1, bernoulli_values(Nat.1), Nat.0) = Rat.1
    Nat.1 <= Nat.1
    bernoulli_summand(Nat.1, bernoulli_values(Nat.1), Nat.1) =
        Rat.from_nat(Nat.3.binom(Nat.1)) * bernoulli(Nat.1)
    Nat.3.binom(Nat.1) = Nat.3
    Rat.from_nat(Nat.3) * (-Rat.1 / Rat.2) = -Rat.3 / Rat.2
    bernoulli_summand(Nat.1, bernoulli_values(Nat.1), Nat.1) = -Rat.3 / Rat.2
    -Rat.3 = Rat.from_int(-Int.3)
    -Rat.3 / Rat.2 = Rat.from_int(-Int.3) / Rat.from_int(Int.2)
    bernoulli_div_from_int(Rat.from_int(-Int.3), Int.2)
    Rat.from_int(-Int.3) / Rat.from_int(Int.2) = reduce(-Int.3, Int.2)
    -Rat.3 / Rat.2 = reduce(-Int.3, Int.2)
    Rat.1 = reduce(Int.2, Int.2)
    add_reduced_same_denom(Int.2, -Int.3, Int.2)
    reduce(Int.2, Int.2) + reduce(-Int.3, Int.2) = reduce(Int.2 + -Int.3, Int.2)
    Int.2 = sub_nat(Nat.2, Nat.0)
    -Int.3 = sub_nat(Nat.0, Nat.3)
    add_sub_nat(Nat.2, Nat.0, Nat.0, Nat.3)
    sub_nat(Nat.2, Nat.0) + sub_nat(Nat.0, Nat.3) = sub_nat(Nat.2 + Nat.0, Nat.0 + Nat.3)
    add_zero_right(Nat.2)
    add_zero_left(Nat.3)
    sub_nat(Nat.2 + Nat.0, Nat.0 + Nat.3) = sub_nat(Nat.2, Nat.3)
    sub_nat(Nat.2, Nat.3) = -Int.1
    Int.2 + -Int.3 = -Int.1
    reduce(Int.2 + -Int.3, Int.2) = reduce(-Int.1, Int.2)
    -Rat.1 = Rat.from_int(-Int.1)
    -Rat.1 / Rat.2 = Rat.from_int(-Int.1) / Rat.from_int(Int.2)
    bernoulli_div_from_int(Rat.from_int(-Int.1), Int.2)
    Rat.from_int(-Int.1) / Rat.from_int(Int.2) = reduce(-Int.1, Int.2)
    -Rat.1 / Rat.2 = reduce(-Int.1, Int.2)
    Rat.1 + (-Rat.3 / Rat.2) = -Rat.1 / Rat.2
    partial(bernoulli_summand(Nat.1, bernoulli_values(Nat.1)), Nat.2) = -Rat.1 / Rat.2
    bernoulli(Nat.2) = -Rat.1 / Rat.3 * (-Rat.1 / Rat.2)
    mul_fractions(-Rat.1, Rat.3, -Rat.1, Rat.2)
    (-Rat.1 / Rat.3) * (-Rat.1 / Rat.2) = (-Rat.1 * -Rat.1) / (Rat.3 * Rat.2)
    (-Rat.1) * (-Rat.1) = Rat.1
    Rat.3 = Rat.from_int(Int.3)
    Rat.2 = Rat.from_int(Int.2)
    mul_int_eq_int_mul(Int.3, Int.2)
    Rat.from_int(Int.3) * Rat.from_int(Int.2) = Rat.from_int(Int.3 * Int.2)
    Int.3 * Int.2 = Int.6
    Rat.from_int(Int.6) = Rat.6
    Rat.3 * Rat.2 = Rat.6
    (-Rat.1 / Rat.3) * (-Rat.1 / Rat.2) = Rat.1 / Rat.6
    bernoulli(Nat.2) = Rat.1 / Rat.6
}

/// Two plus negative four is negative two.
theorem bernoulli_int_two_add_neg_four {
    Int.2 + -Int.4 = -Int.2
} by {
    Int.2 = sub_nat(Nat.2, Nat.0)
    -Int.4 = sub_nat(Nat.0, Nat.4)
    add_sub_nat(Nat.2, Nat.0, Nat.0, Nat.4)
    sub_nat(Nat.2, Nat.0) + sub_nat(Nat.0, Nat.4) = sub_nat(Nat.2 + Nat.0, Nat.0 + Nat.4)
    add_zero_right(Nat.2)
    add_zero_left(Nat.4)
    sub_nat(Nat.2 + Nat.0, Nat.0 + Nat.4) = sub_nat(Nat.2, Nat.4)
    Nat.2 + Nat.2 = Nat.4
    add_imp_sub(Nat.2, Nat.2, Nat.4)
    Nat.4 - Nat.2 = Nat.2
    Nat.2 < Nat.4
    lte_imp_not_lt(Nat.4, Nat.2)
    not Nat.4 <= Nat.2
    sub_nat(Nat.2, Nat.4) = -(Int.from_nat(Nat.4 - Nat.2))
    -(Int.from_nat(Nat.2)) = -Int.2
    sub_nat(Nat.2, Nat.4) = -Int.2
    Int.2 + -Int.4 = -Int.2
}

/// Two plus negative five is negative three.
theorem bernoulli_int_two_add_neg_five {
    Int.2 + -Int.5 = -Int.3
} by {
    Int.2 = sub_nat(Nat.2, Nat.0)
    -Int.5 = sub_nat(Nat.0, Nat.5)
    add_sub_nat(Nat.2, Nat.0, Nat.0, Nat.5)
    sub_nat(Nat.2, Nat.0) + sub_nat(Nat.0, Nat.5) = sub_nat(Nat.2 + Nat.0, Nat.0 + Nat.5)
    add_zero_right(Nat.2)
    add_zero_left(Nat.5)
    sub_nat(Nat.2 + Nat.0, Nat.0 + Nat.5) = sub_nat(Nat.2, Nat.5)
    Nat.2 + Nat.3 = Nat.5
    add_imp_sub(Nat.2, Nat.3, Nat.5)
    Nat.5 - Nat.2 = Nat.3
    Nat.2 < Nat.5
    lte_imp_not_lt(Nat.5, Nat.2)
    not Nat.5 <= Nat.2
    sub_nat(Nat.2, Nat.5) = -(Int.from_nat(Nat.5 - Nat.2))
    -(Int.from_nat(Nat.3)) = -Int.3
    sub_nat(Nat.2, Nat.5) = -Int.3
    Int.2 + -Int.5 = -Int.3
}

/// Two plus negative six is negative four.
theorem bernoulli_int_two_add_neg_six {
    Int.2 + -Int.6 = -Int.4
} by {
    Int.2 = sub_nat(Nat.2, Nat.0)
    -Int.6 = sub_nat(Nat.0, Nat.6)
    add_sub_nat(Nat.2, Nat.0, Nat.0, Nat.6)
    sub_nat(Nat.2, Nat.0) + sub_nat(Nat.0, Nat.6) = sub_nat(Nat.2 + Nat.0, Nat.0 + Nat.6)
    add_zero_right(Nat.2)
    add_zero_left(Nat.6)
    sub_nat(Nat.2 + Nat.0, Nat.0 + Nat.6) = sub_nat(Nat.2, Nat.6)
    Nat.2 + Nat.4 = Nat.6
    add_imp_sub(Nat.2, Nat.4, Nat.6)
    Nat.6 - Nat.2 = Nat.4
    Nat.2 < Nat.6
    lte_imp_not_lt(Nat.6, Nat.2)
    not Nat.6 <= Nat.2
    sub_nat(Nat.2, Nat.6) = -(Int.from_nat(Nat.6 - Nat.2))
    -(Int.from_nat(Nat.4)) = -Int.4
    sub_nat(Nat.2, Nat.6) = -Int.4
    Int.2 + -Int.6 = -Int.4
}

/// Negative nine plus ten is one.
theorem bernoulli_int_neg_nine_add_ten {
    -Int.9 + Int.10 = Int.1
} by {
    -Int.9 = sub_nat(Nat.0, Nat.9)
    Int.10 = sub_nat(Nat.10, Nat.0)
    add_sub_nat(Nat.0, Nat.9, Nat.10, Nat.0)
    sub_nat(Nat.0, Nat.9) + sub_nat(Nat.10, Nat.0) = sub_nat(Nat.0 + Nat.10, Nat.9 + Nat.0)
    add_zero_left(Nat.10)
    add_zero_right(Nat.9)
    sub_nat(Nat.0 + Nat.10, Nat.9 + Nat.0) = sub_nat(Nat.10, Nat.9)
    Nat.9 + Nat.1 = Nat.10
    add_imp_sub(Nat.9, Nat.1, Nat.10)
    Nat.10 - Nat.9 = Nat.1
    sub_nat(Nat.10, Nat.9) = Int.from_nat(Nat.10 - Nat.9)
    Int.from_nat(Nat.1) = Int.1
    sub_nat(Nat.10, Nat.9) = Int.1
    -Int.9 + Int.10 = Int.1
}

/// Negative four plus five is one.
theorem bernoulli_int_neg_four_add_five {
    -Int.4 + Int.5 = Int.1
} by {
    -Int.4 = sub_nat(Nat.0, Nat.4)
    Int.5 = sub_nat(Nat.5, Nat.0)
    add_sub_nat(Nat.0, Nat.4, Nat.5, Nat.0)
    sub_nat(Nat.0, Nat.4) + sub_nat(Nat.5, Nat.0) = sub_nat(Nat.0 + Nat.5, Nat.4 + Nat.0)
    add_zero_left(Nat.5)
    add_zero_right(Nat.4)
    sub_nat(Nat.0 + Nat.5, Nat.4 + Nat.0) = sub_nat(Nat.5, Nat.4)
    Nat.4 + Nat.1 = Nat.5
    add_imp_sub(Nat.4, Nat.1, Nat.5)
    Nat.5 - Nat.4 = Nat.1
    sub_nat(Nat.5, Nat.4) = Int.from_nat(Nat.5 - Nat.4)
    Int.from_nat(Nat.1) = Int.1
    sub_nat(Nat.5, Nat.4) = Int.1
    -Int.4 + Int.5 = Int.1
}

/// The third Bernoulli number is zero.
theorem bernoulli_three {
    bernoulli(Nat.3) = Rat.0
} by {
    bernoulli(Nat.3) = bernoulli_next(Nat.2, bernoulli_values(Nat.2))
    bernoulli_next(Nat.2, bernoulli_values(Nat.2)) =
        -Rat.1 / Rat.from_nat(Nat.4) *
        partial(bernoulli_summand(Nat.2, bernoulli_values(Nat.2)), Nat.3)
    Rat.from_nat(Nat.4) = Rat.4
    partial(bernoulli_summand(Nat.2, bernoulli_values(Nat.2)), Nat.3) =
        partial(bernoulli_summand(Nat.2, bernoulli_values(Nat.2)), Nat.2) +
        bernoulli_summand(Nat.2, bernoulli_values(Nat.2), Nat.2)
    partial(bernoulli_summand(Nat.2, bernoulli_values(Nat.2)), Nat.2) =
        partial(bernoulli_summand(Nat.2, bernoulli_values(Nat.2)), Nat.1) +
        bernoulli_summand(Nat.2, bernoulli_values(Nat.2), Nat.1)
    partial(bernoulli_summand(Nat.2, bernoulli_values(Nat.2)), Nat.1) =
        bernoulli_summand(Nat.2, bernoulli_values(Nat.2), Nat.0)
    Nat.0 <= Nat.2
    bernoulli_summand(Nat.2, bernoulli_values(Nat.2), Nat.0) =
        Rat.from_nat(Nat.4.binom(Nat.0)) * bernoulli(Nat.0)
    Nat.4.binom(Nat.0) = Nat.1
    Rat.from_nat(Nat.1) * Rat.1 = Rat.1
    bernoulli_summand(Nat.2, bernoulli_values(Nat.2), Nat.0) = Rat.1
    Rat.1 = reduce(Int.2, Int.2)
    bernoulli_summand(Nat.2, bernoulli_values(Nat.2), Nat.0) = reduce(Int.2, Int.2)
    Nat.1 <= Nat.2
    bernoulli_summand(Nat.2, bernoulli_values(Nat.2), Nat.1) =
        Rat.from_nat(Nat.4.binom(Nat.1)) * bernoulli(Nat.1)
    Nat.4.binom(Nat.1) = Nat.4
    Rat.from_nat(Nat.4) * (-Rat.1 / Rat.2) = -Rat.4 / Rat.2
    bernoulli_summand(Nat.2, bernoulli_values(Nat.2), Nat.1) = -Rat.4 / Rat.2
    -Rat.4 = Rat.from_int(-Int.4)
    -Rat.4 / Rat.2 = Rat.from_int(-Int.4) / Rat.from_int(Int.2)
    bernoulli_div_from_int(Rat.from_int(-Int.4), Int.2)
    Rat.from_int(-Int.4) / Rat.from_int(Int.2) = reduce(-Int.4, Int.2)
    -Rat.4 / Rat.2 = reduce(-Int.4, Int.2)
    bernoulli_summand(Nat.2, bernoulli_values(Nat.2), Nat.1) = reduce(-Int.4, Int.2)
    Nat.2 <= Nat.2
    bernoulli_summand(Nat.2, bernoulli_values(Nat.2), Nat.2) =
        Rat.from_nat(Nat.4.binom(Nat.2)) * bernoulli(Nat.2)
    Nat.2 + Nat.2 = Nat.4
    exists(c: Nat) {
        Nat.2 + c = Nat.4
    }
    Nat.2 <= Nat.4
    binom_two_double(Nat.4)
    Nat.2 * Nat.4.binom(Nat.2) = Nat.4 * (Nat.4 - Nat.1)
    Nat.4 - Nat.1 = Nat.3
    Nat.4 * Nat.3 = Nat.12
    Nat.2 * Nat.4.binom(Nat.2) = Nat.12
    Nat.2 * Nat.6 = Nat.12
    Nat.2 * Nat.4.binom(Nat.2) = Nat.2 * Nat.6
    Nat.2 != Nat.0
    mul_cancel_left(Nat.2, Nat.4.binom(Nat.2), Nat.6)
    Nat.4.binom(Nat.2) = Nat.6
    Rat.6 = Rat.from_int(Int.6)
    Rat.1 / Rat.6 = Rat.1 / Rat.from_int(Int.6)
    bernoulli_div_from_int(Rat.1, Int.6)
    Rat.1 / Rat.from_int(Int.6) = reduce(Rat.1.num, Rat.1.denom * Int.6)
    Rat.1.num = Int.1
    Rat.1.denom = Int.1
    Rat.1.denom * Int.6 = Int.6
    reduce(Rat.1.num, Rat.1.denom * Int.6) = reduce(Int.1, Int.6)
    Rat.1 / Rat.from_int(Int.6) = reduce(Int.1, Int.6)
    Rat.1 / Rat.6 = reduce(Int.1, Int.6)
    Rat.from_nat(Nat.6) = Rat.from_int(Int.6)
    bernoulli_int_over_reduce(Int.6, Int.6)
    Rat.from_int(Int.6) * reduce(Int.1, Int.6) = reduce(Int.6, Int.6)
    Int.6 = Int.from_nat(Nat.6)
    Int.0 = Int.from_nat(Nat.0)
    if Int.6 = Int.0 {
        Int.from_nat(Nat.6) = Int.from_nat(Nat.0)
        int_from_nat_injective(Nat.6, Nat.0)
        Nat.6 = Nat.0
        false
    }
    Int.6 != Int.0
    reduce_self(Int.6)
    reduce(Int.6, Int.6) = Rat.1
    Rat.from_nat(Nat.6) * (Rat.1 / Rat.6) = Rat.1
    bernoulli_summand(Nat.2, bernoulli_values(Nat.2), Nat.2) = Rat.1
    Rat.1 = reduce(Int.2, Int.2)
    bernoulli_summand(Nat.2, bernoulli_values(Nat.2), Nat.2) = reduce(Int.2, Int.2)
    add_reduced_same_denom(Int.2, -Int.4, Int.2)
    reduce(Int.2, Int.2) + reduce(-Int.4, Int.2) = reduce(Int.2 + -Int.4, Int.2)
    bernoulli_int_two_add_neg_four
    Int.2 + -Int.4 = -Int.2
    reduce(Int.2 + -Int.4, Int.2) = reduce(-Int.2, Int.2)
    add_reduced_same_denom(-Int.2, Int.2, Int.2)
    reduce(-Int.2, Int.2) + reduce(Int.2, Int.2) = reduce(-Int.2 + Int.2, Int.2)
    -Int.2 + Int.2 = Int.0
    reduce(-Int.2 + Int.2, Int.2) = reduce(Int.0, Int.2)
    reduce(Int.0, Int.2) = Rat.0
    partial(bernoulli_summand(Nat.2, bernoulli_values(Nat.2)), Nat.3) = Rat.0
    bernoulli(Nat.3) = -Rat.1 / Rat.4 * Rat.0
    -Rat.1 / Rat.4 * Rat.0 = Rat.0
    bernoulli(Nat.3) = Rat.0
}


/// A natural is at most any larger natural with the right difference.
theorem bernoulli_lte_witness(m: Nat, c: Nat, n: Nat) {
    m + c = n implies m <= n
} by {
    if m + c = n {
        exists(k: Nat) {
            m + k = n
        }
        m <= n
    }
}

/// Five choose two is ten.
theorem bernoulli_binom_five_two {
    Nat.5.binom(Nat.2) = Nat.10
} by {
    Nat.2 + Nat.3 = Nat.5
    exists(c: Nat) {
        Nat.2 + c = Nat.5
    }
    Nat.2 <= Nat.5
    binom_two_double(Nat.5)
    Nat.2 * Nat.5.binom(Nat.2) = Nat.5 * (Nat.5 - Nat.1)
    Nat.5 - Nat.1 = Nat.4
    Nat.5 * Nat.4 = Nat.20
    Nat.2 * Nat.5.binom(Nat.2) = Nat.20
    Nat.2 * Nat.10 = Nat.20
    Nat.2 * Nat.5.binom(Nat.2) = Nat.2 * Nat.10
    Nat.2 != Nat.0
    mul_cancel_left(Nat.2, Nat.5.binom(Nat.2), Nat.10)
    Nat.5.binom(Nat.2) = Nat.10
}

/// Six choose two is fifteen.
theorem bernoulli_binom_six_two {
    Nat.6.binom(Nat.2) = Nat.15
} by {
    pascal_suc_unbounded(Nat.5, Nat.1)
    Nat.6.binom(Nat.2) = Nat.5.binom(Nat.1) + Nat.5.binom(Nat.2)
    choose_one(Nat.5)
    Nat.5.binom(Nat.1) = Nat.5
    bernoulli_binom_five_two
    Nat.5.binom(Nat.2) = Nat.10
    Nat.5 + Nat.10 = Nat.15
    Nat.6.binom(Nat.2) = Nat.15
}

/// Six choose three is twenty.
theorem bernoulli_binom_six_three {
    Nat.6.binom(Nat.3) = Nat.20
} by {
    pascal_suc_unbounded(Nat.5, Nat.2)
    Nat.6.binom(Nat.3) = Nat.5.binom(Nat.2) + Nat.5.binom(Nat.3)
    bernoulli_binom_five_two
    Nat.5.binom(Nat.2) = Nat.10
    Nat.3 + Nat.2 = Nat.5
    add_imp_sub_left(Nat.3, Nat.2, Nat.5)
    Nat.5 - Nat.3 = Nat.2
    exists(c: Nat) {
        Nat.3 + c = Nat.5
    }
    Nat.3 <= Nat.5
    choose_symm_sub_form(Nat.5, Nat.3)
    Nat.5.binom(Nat.3) = Nat.5.binom(Nat.5 - Nat.3)
    Nat.5.binom(Nat.3) = Nat.5.binom(Nat.2)
    Nat.5.binom(Nat.3) = Nat.10
    Nat.10 + Nat.10 = Nat.20
    Nat.6.binom(Nat.3) = Nat.20
}

/// Six choose four is fifteen.
theorem bernoulli_binom_six_four {
    Nat.6.binom(Nat.4) = Nat.15
} by {
    Nat.4 + Nat.2 = Nat.6
    add_imp_sub_left(Nat.4, Nat.2, Nat.6)
    Nat.6 - Nat.4 = Nat.2
    exists(c: Nat) {
        Nat.4 + c = Nat.6
    }
    Nat.4 <= Nat.6
    choose_symm_sub_form(Nat.6, Nat.4)
    Nat.6.binom(Nat.4) = Nat.6.binom(Nat.6 - Nat.4)
    Nat.6.binom(Nat.4) = Nat.6.binom(Nat.2)
    bernoulli_binom_six_two
    Nat.6.binom(Nat.2) = Nat.15
    Nat.6.binom(Nat.4) = Nat.15
}

/// Fifteen times two is thirty.
theorem bernoulli_int_15_mul_2 {
    Int.15 * Int.2 = Int.from_nat(Nat.30)
} by {
    Nat.5 * Nat.3 = Nat.15
    Nat.15 = Nat.5 * Nat.3
    Int.15 = Int.from_nat(Nat.15)
    Int.2 = Int.from_nat(Nat.2)
    mul_from_nat(Nat.15, Nat.2)
    Int.from_nat(Nat.15) * Int.from_nat(Nat.2) = Int.from_nat(Nat.15 * Nat.2)
    Nat.15 * Nat.2 = (Nat.5 * Nat.3) * Nat.2
    mul_assoc(Nat.5, Nat.3, Nat.2)
    (Nat.5 * Nat.3) * Nat.2 = Nat.5 * (Nat.3 * Nat.2)
    Nat.3 * Nat.2 = Nat.6
    Nat.5 * (Nat.3 * Nat.2) = Nat.5 * Nat.6
    Nat.5 * Nat.6 = Nat.30
    Nat.15 * Nat.2 = Nat.30
    Int.from_nat(Nat.15 * Nat.2) = Int.from_nat(Nat.30)
    Int.15 * Int.2 = Int.from_nat(Nat.30)
}

/// Fifteen times two is thirty, at the level of natural numbers.
theorem bernoulli_nat_15_mul_2 {
    Nat.15 * Nat.2 = Nat.30
} by {
    Nat.5 * Nat.3 = Nat.15
    Nat.15 = Nat.5 * Nat.3
    Nat.15 * Nat.2 = (Nat.5 * Nat.3) * Nat.2
    mul_assoc(Nat.5, Nat.3, Nat.2)
    (Nat.5 * Nat.3) * Nat.2 = Nat.5 * (Nat.3 * Nat.2)
    Nat.3 * Nat.2 = Nat.6
    Nat.5 * (Nat.3 * Nat.2) = Nat.5 * Nat.6
    Nat.5 * Nat.6 = Nat.30
    Nat.15 * Nat.2 = Nat.30
}

/// Five times six is thirty.
theorem bernoulli_int_5_mul_6 {
    Int.5 * Int.6 = Int.from_nat(Nat.30)
} by {
    Int.5 = Int.from_nat(Nat.5)
    Int.6 = Int.from_nat(Nat.6)
    mul_from_nat(Nat.5, Nat.6)
    Int.from_nat(Nat.5) * Int.from_nat(Nat.6) = Int.from_nat(Nat.5 * Nat.6)
    Nat.5 * Nat.6 = Nat.30
    Int.from_nat(Nat.5 * Nat.6) = Int.from_nat(Nat.30)
    Int.5 * Int.6 = Int.from_nat(Nat.30)
}

/// Three times five is fifteen.
theorem bernoulli_int_3_mul_5 {
    Int.3 * Int.5 = Int.from_nat(Nat.15)
} by {
    Int.3 = Int.from_nat(Nat.3)
    Int.5 = Int.from_nat(Nat.5)
    mul_from_nat(Nat.3, Nat.5)
    Int.from_nat(Nat.3) * Int.from_nat(Nat.5) = Int.from_nat(Nat.3 * Nat.5)
    Nat.3 * Nat.5 = Nat.15
    Int.from_nat(Nat.3 * Nat.5) = Int.from_nat(Nat.15)
    Int.3 * Int.5 = Int.from_nat(Nat.15)
}

/// Two times five is ten.
theorem bernoulli_int_2_mul_5 {
    Int.2 * Int.5 = Int.from_nat(Nat.10)
} by {
    Int.2 = Int.from_nat(Nat.2)
    Int.5 = Int.from_nat(Nat.5)
    mul_from_nat(Nat.2, Nat.5)
    Int.from_nat(Nat.2) * Int.from_nat(Nat.5) = Int.from_nat(Nat.2 * Nat.5)
    Nat.2 * Nat.5 = Nat.10
    Int.from_nat(Nat.2 * Nat.5) = Int.from_nat(Nat.10)
    Int.2 * Int.5 = Int.from_nat(Nat.10)
}

/// The fourth Bernoulli number is minus one thirtieth.
theorem bernoulli_four {
    bernoulli(Nat.4) = -Rat.1 / Rat.from_nat(Nat.30)
} by {
    bernoulli(Nat.4) = bernoulli_next(Nat.3, bernoulli_values(Nat.3))
    bernoulli_next(Nat.3, bernoulli_values(Nat.3)) =
        -Rat.1 / Rat.from_nat(Nat.5) *
        partial(bernoulli_summand(Nat.3, bernoulli_values(Nat.3)), Nat.4)
    Rat.from_nat(Nat.5) = Rat.5
    partial(bernoulli_summand(Nat.3, bernoulli_values(Nat.3)), Nat.4) =
        partial(bernoulli_summand(Nat.3, bernoulli_values(Nat.3)), Nat.3) +
        bernoulli_summand(Nat.3, bernoulli_values(Nat.3), Nat.3)
    partial(bernoulli_summand(Nat.3, bernoulli_values(Nat.3)), Nat.3) =
        partial(bernoulli_summand(Nat.3, bernoulli_values(Nat.3)), Nat.2) +
        bernoulli_summand(Nat.3, bernoulli_values(Nat.3), Nat.2)
    partial(bernoulli_summand(Nat.3, bernoulli_values(Nat.3)), Nat.2) =
        partial(bernoulli_summand(Nat.3, bernoulli_values(Nat.3)), Nat.1) +
        bernoulli_summand(Nat.3, bernoulli_values(Nat.3), Nat.1)
    partial(bernoulli_summand(Nat.3, bernoulli_values(Nat.3)), Nat.1) =
        bernoulli_summand(Nat.3, bernoulli_values(Nat.3), Nat.0)
    Nat.0 <= Nat.3
    bernoulli_summand(Nat.3, bernoulli_values(Nat.3), Nat.0) =
        Rat.from_nat(Nat.5.binom(Nat.0)) * bernoulli(Nat.0)
    Nat.5.binom(Nat.0) = Nat.1
    Rat.from_nat(Nat.1) * Rat.1 = Rat.1
    bernoulli_summand(Nat.3, bernoulli_values(Nat.3), Nat.0) = Rat.1
    Rat.1 = reduce(Int.2, Int.2)
    bernoulli_summand(Nat.3, bernoulli_values(Nat.3), Nat.0) = reduce(Int.2, Int.2)
    Nat.1 + Nat.2 = Nat.3
    bernoulli_lte_witness(Nat.1, Nat.2, Nat.3)
    Nat.1 <= Nat.3
    bernoulli_summand(Nat.3, bernoulli_values(Nat.3), Nat.1) =
        Rat.from_nat(Nat.5.binom(Nat.1)) * bernoulli(Nat.1)
    Nat.5.binom(Nat.1) = Nat.5
    Rat.from_nat(Nat.5) * (-Rat.1 / Rat.2) = -Rat.5 / Rat.2
    bernoulli_summand(Nat.3, bernoulli_values(Nat.3), Nat.1) = -Rat.5 / Rat.2
    -Rat.5 = Rat.from_int(-Int.5)
    -Rat.5 / Rat.2 = Rat.from_int(-Int.5) / Rat.from_int(Int.2)
    bernoulli_div_from_int(Rat.from_int(-Int.5), Int.2)
    Rat.from_int(-Int.5) / Rat.from_int(Int.2) = reduce(-Int.5, Int.2)
    -Rat.5 / Rat.2 = reduce(-Int.5, Int.2)
    bernoulli_summand(Nat.3, bernoulli_values(Nat.3), Nat.1) = reduce(-Int.5, Int.2)
    Nat.2 + Nat.1 = Nat.3
    bernoulli_lte_witness(Nat.2, Nat.1, Nat.3)
    Nat.2 <= Nat.3
    bernoulli_summand(Nat.3, bernoulli_values(Nat.3), Nat.2) =
        Rat.from_nat(Nat.5.binom(Nat.2)) * bernoulli(Nat.2)
    bernoulli_binom_five_two
    Nat.5.binom(Nat.2) = Nat.10
    bernoulli(Nat.2) = Rat.1 / Rat.6
    Rat.from_nat(Nat.10) = Rat.from_int(Int.10)
    Rat.1 / Rat.6 = Rat.1 / Rat.from_int(Int.6)
    bernoulli_div_from_int(Rat.1, Int.6)
    Rat.1 / Rat.from_int(Int.6) = reduce(Rat.1.num, Rat.1.denom * Int.6)
    Rat.1.num = Int.1
    Rat.1.denom = Int.1
    Rat.1.denom * Int.6 = Int.6
    reduce(Rat.1.num, Rat.1.denom * Int.6) = reduce(Int.1, Int.6)
    Rat.1 / Rat.6 = reduce(Int.1, Int.6)
    bernoulli_int_over_reduce(Int.10, Int.6)
    Rat.from_int(Int.10) * reduce(Int.1, Int.6) = reduce(Int.10, Int.6)
    bernoulli_summand(Nat.3, bernoulli_values(Nat.3), Nat.2) = reduce(Int.10, Int.6)
    Int.2 != Int.0
    reduce_cancels_left(Int.5, Int.3, Int.2)
    reduce(Int.5, Int.3) = reduce(Int.2 * Int.5, Int.2 * Int.3)
    bernoulli_int_2_mul_5
    Int.2 * Int.5 = Int.from_nat(Nat.10)
    Int.2 * Int.3 = Int.6
    reduce(Int.2 * Int.5, Int.2 * Int.3) = reduce(Int.10, Int.6)
    reduce(Int.10, Int.6) = reduce(Int.5, Int.3)
    bernoulli_summand(Nat.3, bernoulli_values(Nat.3), Nat.2) = reduce(Int.5, Int.3)
    Nat.3 <= Nat.3
    bernoulli_summand(Nat.3, bernoulli_values(Nat.3), Nat.3) =
        Rat.from_nat(Nat.5.binom(Nat.3)) * bernoulli(Nat.3)
    bernoulli(Nat.3) = Rat.0
    Rat.from_nat(Nat.5.binom(Nat.3)) * Rat.0 = Rat.0
    bernoulli_summand(Nat.3, bernoulli_values(Nat.3), Nat.3) = Rat.0
    add_reduced_same_denom(Int.2, -Int.5, Int.2)
    reduce(Int.2, Int.2) + reduce(-Int.5, Int.2) = reduce(Int.2 + -Int.5, Int.2)
    bernoulli_int_two_add_neg_five
    Int.2 + -Int.5 = -Int.3
    reduce(Int.2 + -Int.5, Int.2) = reduce(-Int.3, Int.2)
    add_reduced(-Int.3, Int.2, Int.5, Int.3)
    reduce(-Int.3, Int.2) + reduce(Int.5, Int.3) = reduce(-Int.3 * Int.3 + Int.2 * Int.5, Int.2 * Int.3)
    -Int.3 * Int.3 = -Int.9
    bernoulli_int_2_mul_5
    Int.2 * Int.5 = Int.from_nat(Nat.10)
    bernoulli_int_neg_nine_add_ten
    -Int.9 + Int.10 = Int.1
    Int.2 * Int.3 = Int.6
    reduce(-Int.3 * Int.3 + Int.2 * Int.5, Int.2 * Int.3) = reduce(Int.1, Int.6)
    reduce(-Int.3, Int.2) + reduce(Int.5, Int.3) = reduce(Int.1, Int.6)
    partial(bernoulli_summand(Nat.3, bernoulli_values(Nat.3)), Nat.3) = reduce(Int.1, Int.6)
    reduce(Int.1, Int.6) + Rat.0 = reduce(Int.1, Int.6)
    partial(bernoulli_summand(Nat.3, bernoulli_values(Nat.3)), Nat.4) = reduce(Int.1, Int.6)
    bernoulli(Nat.4) = -Rat.1 / Rat.5 * reduce(Int.1, Int.6)
    reduce(Int.1, Int.6) = Rat.1 / Rat.6
    -Rat.1 / Rat.5 * reduce(Int.1, Int.6) = -Rat.1 / Rat.5 * (Rat.1 / Rat.6)
    mul_fractions(-Rat.1, Rat.5, Rat.1, Rat.6)
    (-Rat.1 / Rat.5) * (Rat.1 / Rat.6) = (-Rat.1 * Rat.1) / (Rat.5 * Rat.6)
    -Rat.1 * Rat.1 = -Rat.1
    Rat.5 = Rat.from_int(Int.5)
    Rat.6 = Rat.from_int(Int.6)
    mul_int_eq_int_mul(Int.5, Int.6)
    Rat.from_int(Int.5) * Rat.from_int(Int.6) = Rat.from_int(Int.5 * Int.6)
    bernoulli_int_5_mul_6
    Int.5 * Int.6 = Int.from_nat(Nat.30)
    Rat.from_int(Int.from_nat(Nat.30)) = Rat.from_nat(Nat.30)
    Rat.5 * Rat.6 = Rat.from_nat(Nat.30)
    (-Rat.1 / Rat.5) * (Rat.1 / Rat.6) = -Rat.1 / Rat.from_nat(Nat.30)
    bernoulli(Nat.4) = -Rat.1 / Rat.from_nat(Nat.30)
}

/// The fifth Bernoulli number is zero.
theorem bernoulli_five {
    bernoulli(Nat.5) = Rat.0
} by {
    bernoulli(Nat.5) = bernoulli_next(Nat.4, bernoulli_values(Nat.4))
    bernoulli_next(Nat.4, bernoulli_values(Nat.4)) =
        -Rat.1 / Rat.from_nat(Nat.6) *
        partial(bernoulli_summand(Nat.4, bernoulli_values(Nat.4)), Nat.5)
    Rat.from_nat(Nat.6) = Rat.6
    partial(bernoulli_summand(Nat.4, bernoulli_values(Nat.4)), Nat.5) =
        partial(bernoulli_summand(Nat.4, bernoulli_values(Nat.4)), Nat.4) +
        bernoulli_summand(Nat.4, bernoulli_values(Nat.4), Nat.4)
    partial(bernoulli_summand(Nat.4, bernoulli_values(Nat.4)), Nat.4) =
        partial(bernoulli_summand(Nat.4, bernoulli_values(Nat.4)), Nat.3) +
        bernoulli_summand(Nat.4, bernoulli_values(Nat.4), Nat.3)
    partial(bernoulli_summand(Nat.4, bernoulli_values(Nat.4)), Nat.3) =
        partial(bernoulli_summand(Nat.4, bernoulli_values(Nat.4)), Nat.2) +
        bernoulli_summand(Nat.4, bernoulli_values(Nat.4), Nat.2)
    partial(bernoulli_summand(Nat.4, bernoulli_values(Nat.4)), Nat.2) =
        partial(bernoulli_summand(Nat.4, bernoulli_values(Nat.4)), Nat.1) +
        bernoulli_summand(Nat.4, bernoulli_values(Nat.4), Nat.1)
    partial(bernoulli_summand(Nat.4, bernoulli_values(Nat.4)), Nat.1) =
        bernoulli_summand(Nat.4, bernoulli_values(Nat.4), Nat.0)
    Nat.0 <= Nat.4
    bernoulli_summand(Nat.4, bernoulli_values(Nat.4), Nat.0) =
        Rat.from_nat(Nat.6.binom(Nat.0)) * bernoulli(Nat.0)
    Nat.6.binom(Nat.0) = Nat.1
    Rat.from_nat(Nat.1) * Rat.1 = Rat.1
    bernoulli_summand(Nat.4, bernoulli_values(Nat.4), Nat.0) = Rat.1
    Rat.1 = reduce(Int.2, Int.2)
    bernoulli_summand(Nat.4, bernoulli_values(Nat.4), Nat.0) = reduce(Int.2, Int.2)
    Nat.1 + Nat.3 = Nat.4
    bernoulli_lte_witness(Nat.1, Nat.3, Nat.4)
    Nat.1 <= Nat.4
    bernoulli_summand(Nat.4, bernoulli_values(Nat.4), Nat.1) =
        Rat.from_nat(Nat.6.binom(Nat.1)) * bernoulli(Nat.1)
    Nat.6.binom(Nat.1) = Nat.6
    Rat.from_nat(Nat.6) * (-Rat.1 / Rat.2) = -Rat.6 / Rat.2
    bernoulli_summand(Nat.4, bernoulli_values(Nat.4), Nat.1) = -Rat.6 / Rat.2
    -Rat.6 = Rat.from_int(-Int.6)
    -Rat.6 / Rat.2 = Rat.from_int(-Int.6) / Rat.from_int(Int.2)
    bernoulli_div_from_int(Rat.from_int(-Int.6), Int.2)
    Rat.from_int(-Int.6) / Rat.from_int(Int.2) = reduce(-Int.6, Int.2)
    -Rat.6 / Rat.2 = reduce(-Int.6, Int.2)
    bernoulli_summand(Nat.4, bernoulli_values(Nat.4), Nat.1) = reduce(-Int.6, Int.2)
    Nat.2 + Nat.2 = Nat.4
    bernoulli_lte_witness(Nat.2, Nat.2, Nat.4)
    Nat.2 <= Nat.4
    bernoulli_summand(Nat.4, bernoulli_values(Nat.4), Nat.2) =
        Rat.from_nat(Nat.6.binom(Nat.2)) * bernoulli(Nat.2)
    bernoulli_binom_six_two
    Nat.6.binom(Nat.2) = Nat.15
    bernoulli(Nat.2) = Rat.1 / Rat.6
    Rat.from_nat(Nat.15) = Rat.from_int(Int.from_nat(Nat.15))
    Rat.1 / Rat.6 = Rat.1 / Rat.from_int(Int.6)
    bernoulli_div_from_int(Rat.1, Int.6)
    Rat.1 / Rat.from_int(Int.6) = reduce(Rat.1.num, Rat.1.denom * Int.6)
    Rat.1.num = Int.1
    Rat.1.denom = Int.1
    Rat.1.denom * Int.6 = Int.6
    reduce(Rat.1.num, Rat.1.denom * Int.6) = reduce(Int.1, Int.6)
    Rat.1 / Rat.6 = reduce(Int.1, Int.6)
    bernoulli_int_over_reduce(Int.from_nat(Nat.15), Int.6)
    Rat.from_int(Int.from_nat(Nat.15)) * reduce(Int.1, Int.6) = reduce(Int.from_nat(Nat.15), Int.6)
    bernoulli_summand(Nat.4, bernoulli_values(Nat.4), Nat.2) = reduce(Int.from_nat(Nat.15), Int.6)
    Int.3 != Int.0
    reduce_cancels_left(Int.5, Int.2, Int.3)
    reduce(Int.5, Int.2) = reduce(Int.3 * Int.5, Int.3 * Int.2)
    bernoulli_int_3_mul_5
    Int.3 * Int.5 = Int.from_nat(Nat.15)
    Int.3 * Int.2 = Int.6
    reduce(Int.3 * Int.5, Int.3 * Int.2) = reduce(Int.from_nat(Nat.15), Int.6)
    reduce(Int.from_nat(Nat.15), Int.6) = reduce(Int.5, Int.2)
    bernoulli_summand(Nat.4, bernoulli_values(Nat.4), Nat.2) = reduce(Int.5, Int.2)
    Nat.3 + Nat.1 = Nat.4
    bernoulli_lte_witness(Nat.3, Nat.1, Nat.4)
    Nat.3 <= Nat.4
    bernoulli_summand(Nat.4, bernoulli_values(Nat.4), Nat.3) =
        Rat.from_nat(Nat.6.binom(Nat.3)) * bernoulli(Nat.3)
    bernoulli_binom_six_three
    Nat.6.binom(Nat.3) = Nat.20
    bernoulli(Nat.3) = Rat.0
    Rat.from_nat(Nat.20) * Rat.0 = Rat.0
    bernoulli_summand(Nat.4, bernoulli_values(Nat.4), Nat.3) = Rat.0
    Nat.4 <= Nat.4
    bernoulli_summand(Nat.4, bernoulli_values(Nat.4), Nat.4) =
        Rat.from_nat(Nat.6.binom(Nat.4)) * bernoulli(Nat.4)
    bernoulli_binom_six_four
    Nat.6.binom(Nat.4) = Nat.15
    bernoulli(Nat.4) = -Rat.1 / Rat.from_nat(Nat.30)
    -Rat.1 / Rat.from_nat(Nat.30) = -Rat.1 / Rat.from_int(Int.from_nat(Nat.30))
    -Rat.1 = Rat.from_int(-Int.1)
    -Rat.1 / Rat.from_int(Int.from_nat(Nat.30)) =
        Rat.from_int(-Int.1) / Rat.from_int(Int.from_nat(Nat.30))
    bernoulli_div_from_int(Rat.from_int(-Int.1), Int.from_nat(Nat.30))
    Rat.from_int(-Int.1) / Rat.from_int(Int.from_nat(Nat.30)) =
        reduce(-Int.1, Int.from_nat(Nat.30))
    Rat.from_nat(Nat.15) = reduce(Int.from_nat(Nat.15), Int.1)
    mul_reduced(Int.from_nat(Nat.15), Int.1, -Int.1, Int.from_nat(Nat.30))
    reduce(Int.from_nat(Nat.15), Int.1) * reduce(-Int.1, Int.from_nat(Nat.30)) =
        reduce(Int.from_nat(Nat.15) * -Int.1, Int.1 * Int.from_nat(Nat.30))
    Int.from_nat(Nat.15) * -Int.1 = -(Int.from_nat(Nat.15))
    Int.1 * Int.from_nat(Nat.30) = Int.from_nat(Nat.30)
    reduce(Int.from_nat(Nat.15) * -Int.1, Int.1 * Int.from_nat(Nat.30)) =
        reduce(-Int.from_nat(Nat.15), Int.from_nat(Nat.30))
    Rat.from_nat(Nat.15) * reduce(-Int.1, Int.from_nat(Nat.30)) =
        reduce(-Int.from_nat(Nat.15), Int.from_nat(Nat.30))
    alt_suc_ne_zero(Nat.14)
    Nat.14.suc != Nat.0
    Nat.14.suc = Nat.15
    Nat.15 != Nat.0
    Int.0 = Int.from_nat(Nat.0)
    if Int.from_nat(Nat.15) = Int.0 {
        Int.from_nat(Nat.15) = Int.from_nat(Nat.0)
        int_from_nat_injective(Nat.15, Nat.0)
        Nat.15 = Nat.0
        Nat.15 != Nat.0
        false
    }
    Int.from_nat(Nat.15) != Int.0
    reduce_cancels_left(-Int.1, Int.2, Int.from_nat(Nat.15))
    reduce(-Int.1, Int.2) =
        reduce(Int.from_nat(Nat.15) * -Int.1, Int.from_nat(Nat.15) * Int.2)
    Int.from_nat(Nat.15) * -Int.1 = -(Int.from_nat(Nat.15))
    bernoulli_nat_15_mul_2
    Nat.15 * Nat.2 = Nat.30
    mul_from_nat(Nat.15, Nat.2)
    Int.from_nat(Nat.15) * Int.from_nat(Nat.2) = Int.from_nat(Nat.15 * Nat.2)
    Int.from_nat(Nat.15 * Nat.2) = Int.from_nat(Nat.30)
    Int.from_nat(Nat.15) * Int.from_nat(Nat.2) = Int.from_nat(Nat.30)
    Int.2 = Int.from_nat(Nat.2)
    Int.from_nat(Nat.15) * Int.2 = Int.from_nat(Nat.30)
    reduce(Int.from_nat(Nat.15) * -Int.1, Int.from_nat(Nat.15) * Int.2) =
        reduce(-Int.from_nat(Nat.15), Int.from_nat(Nat.30))
    reduce(-Int.from_nat(Nat.15), Int.from_nat(Nat.30)) = reduce(-Int.1, Int.2)
    Rat.from_nat(Nat.15) * (-Rat.1 / Rat.from_nat(Nat.30)) = reduce(-Int.1, Int.2)
    bernoulli_summand(Nat.4, bernoulli_values(Nat.4), Nat.4) = reduce(-Int.1, Int.2)
    add_reduced_same_denom(Int.2, -Int.6, Int.2)
    reduce(Int.2, Int.2) + reduce(-Int.6, Int.2) = reduce(Int.2 + -Int.6, Int.2)
    bernoulli_int_two_add_neg_six
    Int.2 + -Int.6 = -Int.4
    reduce(Int.2 + -Int.6, Int.2) = reduce(-Int.4, Int.2)
    add_reduced_same_denom(-Int.4, Int.5, Int.2)
    reduce(-Int.4, Int.2) + reduce(Int.5, Int.2) = reduce(-Int.4 + Int.5, Int.2)
    bernoulli_int_neg_four_add_five
    -Int.4 + Int.5 = Int.1
    reduce(-Int.4 + Int.5, Int.2) = reduce(Int.1, Int.2)
    add_reduced_same_denom(Int.1, -Int.1, Int.2)
    reduce(Int.1, Int.2) + reduce(-Int.1, Int.2) = reduce(Int.1 + -Int.1, Int.2)
    Int.1 + -Int.1 = Int.0
    reduce(Int.1 + -Int.1, Int.2) = reduce(Int.0, Int.2)
    reduce(Int.0, Int.2) = Rat.0
    partial(bernoulli_summand(Nat.4, bernoulli_values(Nat.4)), Nat.5) = Rat.0
    bernoulli(Nat.5) = -Rat.1 / Rat.6 * Rat.0
    -Rat.1 / Rat.6 * Rat.0 = Rat.0
    bernoulli(Nat.5) = Rat.0
}


/// The defining recurrence for the Bernoulli numbers: for n >= 1,
/// sum_{k=0}^{n} binom(n+1, k) B_k = 0.
theorem bernoulli_recurrence(m: Nat) {
    partial(bernoulli_recurrence_term(m.suc), m.suc.suc) = Rat.0
} by {
    forall(k: Nat) {
        if k < m.suc {
            k <= m
            bernoulli_summand_eq_term(m, k)
            bernoulli_summand(m, bernoulli_values(m), k) =
                Rat.from_nat(m.suc.suc.binom(k)) * bernoulli(k)
            bernoulli_recurrence_term(m.suc, k) =
                Rat.from_nat(m.suc.suc.binom(k)) * bernoulli(k)
            bernoulli_summand(m, bernoulli_values(m), k) =
                bernoulli_recurrence_term(m.suc, k)
        }
    }
    partial_pointwise_eq(bernoulli_summand(m, bernoulli_values(m)),
        bernoulli_recurrence_term(m.suc), m.suc)
    partial(bernoulli_summand(m, bernoulli_values(m)), m.suc) =
        partial(bernoulli_recurrence_term(m.suc), m.suc)
    bernoulli(m.suc) = bernoulli_next(m, bernoulli_values(m))
    bernoulli_next(m, bernoulli_values(m)) =
        -Rat.1 / Rat.from_nat(m.suc.suc) *
        partial(bernoulli_summand(m, bernoulli_values(m)), m.suc)
    bernoulli(m.suc) =
        -Rat.1 / Rat.from_nat(m.suc.suc) *
        partial(bernoulli_recurrence_term(m.suc), m.suc)
    partial_split_last(bernoulli_recurrence_term(m.suc), m.suc)
    partial(bernoulli_recurrence_term(m.suc), m.suc.suc) =
        partial(bernoulli_recurrence_term(m.suc), m.suc) +
        bernoulli_recurrence_term(m.suc, m.suc)
    bernoulli_recurrence_term(m.suc, m.suc) =
        Rat.from_nat(m.suc.suc.binom(m.suc)) * bernoulli(m.suc)
    Nat.1 + m.suc = m.suc.suc
    exists(c: Nat) {
        Nat.1 + c = m.suc.suc
    }
    Nat.1 <= m.suc.suc
    binom_n_minus_one(m.suc.suc)
    m.suc.suc.binom(m.suc.suc - Nat.1) = m.suc.suc
    suc_sub_one(m.suc)
    m.suc.suc - Nat.1 = m.suc
    m.suc.suc.binom(m.suc) = m.suc.suc
    bernoulli_recurrence_term(m.suc, m.suc) =
        Rat.from_nat(m.suc.suc) * bernoulli(m.suc)
    Nat.0 < m.suc.suc
    nat_lt_imp_rat_lt(Nat.0, m.suc.suc)
    Rat.from_nat(Nat.0) < Rat.from_nat(m.suc.suc)
    Rat.from_nat(Nat.0) = Rat.0
    Rat.0 < Rat.from_nat(m.suc.suc)
    zero_lt_imp_pos(Rat.from_nat(m.suc.suc))
    Rat.from_nat(m.suc.suc).is_positive
    pos_ne_zero(Rat.from_nat(m.suc.suc))
    Rat.from_nat(m.suc.suc) != Rat.0
    Rat.from_nat(m.suc.suc) * bernoulli(m.suc) =
        Rat.from_nat(m.suc.suc) *
        (-Rat.1 / Rat.from_nat(m.suc.suc) *
            partial(bernoulli_recurrence_term(m.suc), m.suc))
    Rat.from_nat(m.suc.suc) *
        (-Rat.1 / Rat.from_nat(m.suc.suc) *
            partial(bernoulli_recurrence_term(m.suc), m.suc)) =
        (Rat.from_nat(m.suc.suc) * (-Rat.1 / Rat.from_nat(m.suc.suc))) *
            partial(bernoulli_recurrence_term(m.suc), m.suc)
    mul_comm(Rat.from_nat(m.suc.suc), -Rat.1 / Rat.from_nat(m.suc.suc))
    Rat.from_nat(m.suc.suc) * (-Rat.1 / Rat.from_nat(m.suc.suc)) =
        (-Rat.1 / Rat.from_nat(m.suc.suc)) * Rat.from_nat(m.suc.suc)
    mul_cancels_div(-Rat.1, Rat.from_nat(m.suc.suc))
    (-Rat.1 / Rat.from_nat(m.suc.suc)) * Rat.from_nat(m.suc.suc) = -Rat.1
    Rat.from_nat(m.suc.suc) * (-Rat.1 / Rat.from_nat(m.suc.suc)) = -Rat.1
    (Rat.from_nat(m.suc.suc) * (-Rat.1 / Rat.from_nat(m.suc.suc))) *
        partial(bernoulli_recurrence_term(m.suc), m.suc) =
        -Rat.1 * partial(bernoulli_recurrence_term(m.suc), m.suc)
    -Rat.1 * partial(bernoulli_recurrence_term(m.suc), m.suc) =
        -partial(bernoulli_recurrence_term(m.suc), m.suc)
    Rat.from_nat(m.suc.suc) *
        (-Rat.1 / Rat.from_nat(m.suc.suc) *
            partial(bernoulli_recurrence_term(m.suc), m.suc)) =
        -partial(bernoulli_recurrence_term(m.suc), m.suc)
    Rat.from_nat(m.suc.suc) * bernoulli(m.suc) =
        -partial(bernoulli_recurrence_term(m.suc), m.suc)
    bernoulli_recurrence_term(m.suc, m.suc) =
        -partial(bernoulli_recurrence_term(m.suc), m.suc)
    partial(bernoulli_recurrence_term(m.suc), m.suc) +
        bernoulli_recurrence_term(m.suc, m.suc) = Rat.0
    partial(bernoulli_recurrence_term(m.suc), m.suc.suc) = Rat.0
}

// Odd Bernoulli numbers from three onward vanish.
//
// The classical statement is B_{2k+1} = 0 for k >= 1, i.e. every odd index
// beyond one gives zero.  The cleanest proof passes through the exponential
// generating function t/(e^t - 1): after adding t/2 the series becomes even,
// which forces all odd coefficients beyond the linear term to vanish.  A
// purely recurrence-based proof is also possible, but it requires formal
// power series over the rationals, which the library does not provide, so the
// statement is left here as a comment rather than a theorem.
//
// theorem bernoulli_odd_zero(k: Nat) {
//     bernoulli(Nat.2 * k + 1) = Rat.0
// }

// The exponential generating function.
//
// The Bernoulli numbers are the coefficients in the exponential generating
// function t/(e^t - 1) = sum_{n >= 0} B_n t^n / n!.  The library has the
// exponential series over the reals (src/real/exp.ac), but that is a
// convergent series of real numbers, not a formal power series over the
// rationals, so the equality cannot be stated as a theorem here.  Proving it
// would require a theory of formal power series together with the identity
// t * sum_{n >= 0} B_n t^n / n! = e^t - 1, expanded through the binomial
// theorem and the defining recurrence.
