/// Finite fields: the field of residues modulo a prime.
///
/// This file gathers the finite-field statements for Z/pZ in the residue
/// language of this library: congruence modulo the prime `p` is equality of
/// remainders (`congr_mod`, congruence.ac), and modular inverses are
/// developed in modular_inverse.ac. The residue classes are packaged as the
/// type `Zmod[n]` (zmod/base.ac) with commutative-ring instances
/// (zmod_typeclass_instances.ac); there is no `Field` instance because
/// `Zmod[n]` is a field only when `n` is prime. Everything below is therefore
/// stated for prime moduli in the natural-number residue language.
///
/// Contents:
///   - Z/pZ is a field: every nonzero residue has a multiplicative inverse
///     (`zmod_prime_mul_inverse_exists`, `zmod_prime_field_mul_inverse`,
///     `zmod_prime_mod_inv_correct`, `zmod_prime_zero_ne_one`, and the
///     nonzero-residue bridge `zmod_prime_coprime_iff_not_divides`).
///   - The multiplicative group of Z/pZ is cyclic (`is_primitive_root_mod_prime`;
///     the existence theorem is recorded in a comment below).
///   - Fermat's little theorem in field form: `a^(p-1) = 1` for nonzero `a`
///     (`fermat_little_field_form`, `fermat_little_field_nonzero_residue`).
///   - The number of nonzero elements of Z/pZ is `p - 1`
///     (`count_nonzero_residues`, `zmod_prime_nonzero_count`).
///   - Wilson's theorem: `(p-1)! ≡ -1 (mod p)` (`wilson_prime_field_form`).

from nat import Nat
from number_theory.congruence import congr_mod_symm, divides_of_congr_mod_zero
from number_theory.coprime import nat_divides_one_imp_one
from number_theory.modular_inverse import nat_modular_inverse_exists, mod_inv,
    mod_inv_mul_congr_one
from number_theory.totient import coprime_below_prime, not_coprime_imp_divides_prime,
    divides_prime_imp_not_coprime
from number_theory.fermat import fermat_euler, sub_one_pred
from number_theory.fermat_consequences import wilsons_factorial_congr_prime
from number_theory.multiplicative_order import multiplicative_order_mod

// ---------------------------------------------------------------------------
// Z/pZ is a field for prime p.
// ---------------------------------------------------------------------------

/// A prime modulus makes every unit residue class invertible: for `a` coprime
/// to the prime `p`, some natural `b` satisfies `a * b ≡ 1 (mod p)`.
/// Restated from `nat_modular_inverse_exists` (modular_inverse.ac).
theorem zmod_prime_mul_inverse_exists(p: Nat, a: Nat) {
    p.is_prime and a.coprime(p) implies exists(b: Nat) { (a * b).congr_mod(Nat.1, p) }
} by {
    if p.is_prime and a.coprime(p) {
        nat_modular_inverse_exists(a, p)
        exists(b: Nat) { (a * b).congr_mod(Nat.1, p) }
    }
}

/// For prime `p`, an element is a unit modulo `p` precisely when it is a
/// nonzero residue: `a` is coprime to `p` exactly when `p` does not divide
/// `a`.
theorem zmod_prime_coprime_iff_not_divides(p: Nat, a: Nat) {
    p.is_prime implies (a.coprime(p) = not p.divides(a))
} by {
    if p.is_prime {
        if a.coprime(p) {
            if p.divides(a) {
                divides_prime_imp_not_coprime(p, a)
                not a.coprime(p)
                false
            }
        }
        if not p.divides(a) {
            if not a.coprime(p) {
                not_coprime_imp_divides_prime(p, a)
                p.divides(a)
                false
            }
            a.coprime(p)
        }
        (a.coprime(p) = not p.divides(a)) = true
    }
}

/// Every nonzero residue modulo a prime has a multiplicative inverse: for
/// `0 < a < p` with `p` prime, some `b` satisfies `a * b ≡ 1 (mod p)`. This
/// is the field axiom of Z/pZ in residue form.
theorem zmod_prime_field_mul_inverse(p: Nat, a: Nat) {
    p.is_prime and Nat.0 < a and a < p implies exists(b: Nat) { (a * b).congr_mod(Nat.1, p) }
} by {
    if p.is_prime and Nat.0 < a and a < p {
        Nat.1 <= a
        coprime_below_prime(p, a)
        a.coprime(p)
        nat_modular_inverse_exists(a, p)
        exists(b: Nat) { (a * b).congr_mod(Nat.1, p) }
    }
}

/// The defined modular inverse realizes the field inverse in Z/pZ: for `a`
/// coprime to the prime `p`, `a * mod_inv(a, p) ≡ 1 (mod p)`.
theorem zmod_prime_mod_inv_correct(p: Nat, a: Nat) {
    p.is_prime and a.coprime(p) implies (a * mod_inv(a, p)).congr_mod(Nat.1, p)
} by {
    if p.is_prime and a.coprime(p) {
        mod_inv_mul_congr_one(a, p)
        (a * mod_inv(a, p)).congr_mod(Nat.1, p)
    }
}

/// Zero and one are distinct in Z/pZ for a prime `p`: `0` is not congruent
/// to `1` modulo `p`.
theorem zmod_prime_zero_ne_one(p: Nat) {
    p.is_prime implies not (Nat.0.congr_mod(Nat.1, p))
} by {
    if p.is_prime {
        if Nat.0.congr_mod(Nat.1, p) {
            congr_mod_symm(Nat.0, Nat.1, p)
            Nat.1.congr_mod(Nat.0, p)
            divides_of_congr_mod_zero(p, Nat.1)
            p.divides(Nat.1)
            nat_divides_one_imp_one(p)
            p = Nat.1
            Nat.1 < p
            false
        }
    }
}

// ---------------------------------------------------------------------------
// The multiplicative group of Z/pZ is cyclic.
// ---------------------------------------------------------------------------

/// True when `g` is a primitive root modulo the prime `p`: `g` is coprime to
/// `p` and its multiplicative order is exactly `p - 1`, the order of the
/// multiplicative group of the field Z/pZ.
define is_primitive_root_mod_prime(g: Nat, p: Nat) -> Bool {
    p.is_prime and g.coprime(p) and multiplicative_order_mod(g, p) = p - Nat.1
}

// The classical theorem that the multiplicative group of Z/pZ is cyclic —
// every prime `p` has a primitive root — is not yet proved in this library.
// `primitive_root.ac` and `primitive_root_applications.ac` develop the theory
// of primitive roots under the hypothesis that one exists (there, `g` is a
// primitive root modulo the prime `p` exactly when
// `is_primitive_root_mod_prime(g, p)` holds), and `carmichael.ac` records the
// same gap: the library has no existence theorem for elements of
// multiplicative order `p - 1` modulo a prime. The statement is therefore
// recorded here, commented out:
//
// theorem primitive_root_exists_mod_prime(p: Nat) {
//     p.is_prime implies exists(g: Nat) { is_primitive_root_mod_prime(g, p) }
// }

// ---------------------------------------------------------------------------
// Fermat's little theorem in field form.
// ---------------------------------------------------------------------------

/// Fermat's little theorem in field form: for `a` coprime to the prime `p`
/// (that is, `a` is a nonzero element of Z/pZ), `a^(p - 1) ≡ 1 (mod p)`.
/// This is `fermat_euler` from fermat.ac, restated for the field.
theorem fermat_little_field_form(p: Nat, a: Nat) {
    p.is_prime and a.coprime(p) implies a.pow(p - Nat.1).congr_mod(Nat.1, p)
} by {
    if p.is_prime and a.coprime(p) {
        fermat_euler(p, a)
        a.pow(p - Nat.1).congr_mod(Nat.1, p)
    }
}

/// Fermat's little theorem for a nonzero residue: for `0 < a < p` with `p`
/// prime, `a^(p - 1) ≡ 1 (mod p)`.
theorem fermat_little_field_nonzero_residue(p: Nat, a: Nat) {
    p.is_prime and Nat.0 < a and a < p implies a.pow(p - Nat.1).congr_mod(Nat.1, p)
} by {
    if p.is_prime and Nat.0 < a and a < p {
        Nat.1 <= a
        coprime_below_prime(p, a)
        a.coprime(p)
        fermat_euler(p, a)
        a.pow(p - Nat.1).congr_mod(Nat.1, p)
    }
}

// ---------------------------------------------------------------------------
// The number of nonzero elements of Z/pZ.
// ---------------------------------------------------------------------------

/// The number of nonzero elements among the residues `0, 1, ..., k - 1`.
/// Written with nested pattern matches (no boolean condition) so that the
/// defining equations reduce by computation.
define count_nonzero_residues(k: Nat) -> Nat {
    match k {
        Nat.zero {
            Nat.0
        }
        Nat.suc(j) {
            match j {
                Nat.zero {
                    Nat.0
                }
                Nat.suc(jj) {
                    count_nonzero_residues(j) + Nat.1
                }
            }
        }
    }
}

/// The residue zero is not counted below one.
theorem count_nonzero_residues_suc_zero {
    count_nonzero_residues(Nat.0.suc) = Nat.0
} by {
    count_nonzero_residues(Nat.0.suc) = Nat.0
}

/// The number of nonzero residues below `j.suc` is exactly `j`: inductively,
/// every bound adds the top residue `j` itself, which is nonzero.
theorem count_nonzero_residues_suc_eq(j: Nat) {
    count_nonzero_residues(j.suc) = j
} by {
    define f(x: Nat) -> Bool { count_nonzero_residues(x.suc) = x }
    count_nonzero_residues(Nat.0.suc) = Nat.0
    f(Nat.0)
    forall(x: Nat) {
        if f(x) {
            count_nonzero_residues(x.suc.suc) = count_nonzero_residues(x.suc) + Nat.1
            count_nonzero_residues(x.suc) = x
            count_nonzero_residues(x.suc.suc) = x + Nat.1
            x + Nat.1 = x.suc
            count_nonzero_residues(x.suc.suc) = x.suc
            f(x.suc)
        }
    }
    f(j)
}

/// A positive bound `k` has exactly `k - 1` nonzero residues below it.
theorem count_nonzero_residues_eq_pred(k: Nat) {
    k != Nat.0 implies count_nonzero_residues(k) = k - Nat.1
} by {
    if k != Nat.0 {
        let pred: Nat satisfy { pred.suc = k }
        count_nonzero_residues_suc_eq(pred)
        count_nonzero_residues(pred.suc) = pred
        k = pred.suc
        count_nonzero_residues(k) = pred
        sub_one_pred(k, pred)
        k - Nat.1 = pred
        count_nonzero_residues(k) = k - Nat.1
    }
}

/// The field Z/pZ has exactly `p - 1` nonzero elements.
theorem zmod_prime_nonzero_count(p: Nat) {
    p.is_prime implies count_nonzero_residues(p) = p - Nat.1
} by {
    if p.is_prime {
        Nat.1 < p
        p != Nat.0
        count_nonzero_residues_eq_pred(p)
        count_nonzero_residues(p) = p - Nat.1
    }
}

// ---------------------------------------------------------------------------
// Wilson's theorem.
// ---------------------------------------------------------------------------

/// Wilson's theorem: for a prime `p`, `(p - 1)! ≡ p - 1 (mod p)`. Since
/// `p - 1` is the canonical representative of `-1` modulo `p`, this is the
/// classical statement `(p - 1)! ≡ -1 (mod p)`. Restated from
/// `wilsons_factorial_congr_prime` (fermat_consequences.ac).
theorem wilson_prime_field_form(p: Nat) {
    p.is_prime implies (p - Nat.1).factorial.congr_mod(p - Nat.1, p)
} by {
    if p.is_prime {
        wilsons_factorial_congr_prime(p)
        (p - Nat.1).factorial.congr_mod(p - Nat.1, p)
    }
}
