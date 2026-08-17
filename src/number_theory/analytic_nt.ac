// Analytic number theory connections: the prime counting function, the Chebyshev functions,
// the Mertens function, and their elementary relationships.
//
// This file connects the Chebyshev functions (`chebyshev.ac`), the von Mangoldt function
// (`von_mangoldt.ac`), and the prime counting function defined below.  The targets:
//
//   (a) `psi(x) = sum_{n <= x} Lambda(n)`, and `theta(x) <= psi(x)`;   [restated from chebyshev.ac]
//   (b) the prime-counting connection: `theta(x) = sum_{p <= x} log p` together with the crude
//       bounds `pi(x) * log 2 <= theta(x) <= pi(x) * log x`;                        [proved]
//   (c) the prime number theorem `pi(x) ~ x / log x` is equivalent to `psi(x) ~ x`;  [deep; comment]
//   (e) from a prime between `n` and `2n`, `pi(2n) - pi(n) >= 1`; the small cases of
//       Bertrand's postulate in counting form.                              [bridge proved]
//
// The Mertens function (target (d), `M(x) = sum_{n <= x} mu(n)`) lives in `mertens_nt.ac`:
// the module graph of `chebyshev.ac` and the module graph of `mobius_inversion.ac` each
// define the `Nat.congr_mod` attribute (through `interface.ac` and `congruence.ac`
// respectively), so no single module can import both files.

from nat import Nat, from_nat, lt_imp_lte_suc, lt_suc_right, lt_suc, lt_trans,
    add_comm, add_assoc, lte_ref, not_lt_zero, lte_and_lt, lt_not_ref,
    lte_imp_not_lt, lt_or_lte, lte_mul_both, divides_sub, divides_self,
    add_sub, lt_add_left
from real import Real, lte_trans, add_lte_add
from order import lt_imp_lte
from data.nat.nat_range_sum import range_sum, range_sum_zero, range_sum_suc, range_sum_congr,
    range_sum_split, shift_fn
from number_theory.chebyshev import chebyshev_theta, chebyshev_theta_weight, chebyshev_psi,
    chebyshev_psi_eq_range_sum_von_mangoldt, chebyshev_theta_lte_psi, log_value_monotone_nat,
    range_sum_lte_real
from number_theory.von_mangoldt import von_mangoldt
from number_theory.falling_product import nat_two_prime
from number_theory.goldbach import five_is_prime, seven_is_prime
from number_theory.factorisation import no_proper_divisor_imp_prime, prime_does_not_divide_one
numerals Nat
numerals Real

// ---------------------------------------------------------------------------
// The prime counting function
// ---------------------------------------------------------------------------

/// One at primes and zero elsewhere, as a real.
define prime_indicator(i: Nat) -> Real {
    if i.is_prime {
        Real.1
    } else {
        Real.0
    }
}

/// The prime counting function `pi(x)`: the number of primes at most `x`.
///
/// The count is embedded in the reals so that the comparison with the Chebyshev functions is
/// a comparison of reals.
define prime_count(x: Nat) -> Real {
    range_sum(prime_indicator, x.suc)
}

/// The indicator of a prime is one.
theorem prime_indicator_of_prime(p: Nat) {
    p.is_prime implies prime_indicator(p) = Real.1
} by {
    if p.is_prime {
        prime_indicator(p) = if p.is_prime { Real.1 } else { Real.0 }
        prime_indicator(p) = Real.1
    }
}

/// The indicator is nonnegative everywhere.
theorem prime_indicator_nonneg(i: Nat) {
    Real.0 <= prime_indicator(i)
} by {
    if i.is_prime {
        prime_indicator(i) = if i.is_prime { Real.1 } else { Real.0 }
        prime_indicator(i) = Real.1
        Real.1 > Real.0
        lt_imp_lte(Real.0, Real.1)
        Real.0 <= Real.1
        Real.0 <= prime_indicator(i)
    }
    if not i.is_prime {
        prime_indicator(i) = if i.is_prime { Real.1 } else { Real.0 }
        prime_indicator(i) = Real.0
        Real.0 <= Real.0
        Real.0 <= prime_indicator(i)
    }
    Real.0 <= prime_indicator(i)
}

// ---------------------------------------------------------------------------
// (a) The von Mangoldt identity and the comparison of the Chebyshev functions
// ---------------------------------------------------------------------------

/// Chebyshev's psi function is the sum of the von Mangoldt function:
/// `psi(x) = sum_{n <= x} Lambda(n)`.
///
/// Proved in `chebyshev.ac`; restated here so that the analytic number theory layer stands
/// alone.
theorem psi_eq_sum_von_mangoldt(x: Nat) {
    chebyshev_psi(x) = range_sum(von_mangoldt, x.suc)
} by {
    chebyshev_psi_eq_range_sum_von_mangoldt(x)
    chebyshev_psi(x) = range_sum(von_mangoldt, x.suc)
}

/// Chebyshev's theta function is at most Chebyshev's psi function: `theta(x) <= psi(x)`.
///
/// Every prime is a prime power, so the theta summand is the psi summand at the primes and
/// zero elsewhere; proved in `chebyshev.ac`.
theorem theta_lte_psi(x: Nat) {
    chebyshev_theta(x) <= chebyshev_psi(x)
} by {
    chebyshev_theta_lte_psi(x)
    chebyshev_theta(x) <= chebyshev_psi(x)
}

// ---------------------------------------------------------------------------
// (b) The prime-counting connection: theta(x) = sum_{p <= x} log p, and the crude bounds
// ---------------------------------------------------------------------------

/// Chebyshev's theta function is the sum of `log p` over the primes at most `x`.
///
/// This is the defining equation of `chebyshev_theta` in `chebyshev.ac`; it is restated here
/// as the bridge between the analytic number theory layer and the prime counting function.
theorem theta_eq_range_sum_weight(x: Nat) {
    chebyshev_theta(x) = range_sum(chebyshev_theta_weight, x.suc)
} by {
    chebyshev_theta(x) = range_sum(chebyshev_theta_weight, x.suc)
}

/// A real-valued summand scaled by a constant.
define mul_const_fn(f: Nat -> Real, c: Real, i: Nat) -> Real {
    f(i) * c
}

/// A range sum of a constant multiple is the constant times the range sum.
theorem range_sum_mul_const_real(f: Nat -> Real, c: Real, n: Nat) {
    range_sum(mul_const_fn(f, c), n) = range_sum(f, n) * c
} by {
    define p(x: Nat) -> Bool {
        range_sum(mul_const_fn(f, c), x) = range_sum(f, x) * c
    }
    range_sum_zero(mul_const_fn(f, c))
    range_sum(mul_const_fn(f, c), Nat.0) = Real.0
    range_sum_zero(f)
    range_sum(f, Nat.0) = Real.0
    Real.0 * c = Real.0
    range_sum(f, Nat.0) * c = Real.0
    range_sum(mul_const_fn(f, c), Nat.0) = range_sum(f, Nat.0) * c
    p(Nat.0)
    forall(k: Nat) {
        if p(k) {
            range_sum(mul_const_fn(f, c), k) = range_sum(f, k) * c
            range_sum_suc(mul_const_fn(f, c), k)
            range_sum(mul_const_fn(f, c), k.suc) =
                range_sum(mul_const_fn(f, c), k) + mul_const_fn(f, c, k)
            mul_const_fn(f, c, k) = f(k) * c
            range_sum(mul_const_fn(f, c), k.suc) = range_sum(f, k) * c + f(k) * c
            range_sum_suc(f, k)
            range_sum(f, k.suc) = range_sum(f, k) + f(k)
            range_sum(f, k) * c + f(k) * c = (range_sum(f, k) + f(k)) * c
            range_sum(mul_const_fn(f, c), k.suc) = range_sum(f, k.suc) * c
            p(k.suc) = (range_sum(mul_const_fn(f, c), k.suc) = range_sum(f, k.suc) * c)
            p(k.suc)
        }
        (p(k) implies p(k.suc))
    }
    p(Nat.0) and forall(k: Nat) {
        p(k) implies p(k.suc)
    }
    Nat.induction(p)
    p(n)
}

/// Range sums of reals are monotone in the summand, pointwise below the limit.
theorem range_sum_lte_real_on_range(f: Nat -> Real, g: Nat -> Real, n: Nat) {
    (forall(i: Nat) { i < n implies f(i) <= g(i) }) implies range_sum(f, n) <= range_sum(g, n)
} by {
    define p(x: Nat) -> Bool {
        (forall(i: Nat) { i < x implies f(i) <= g(i) }) implies range_sum(f, x) <= range_sum(g, x)
    }
    range_sum_zero(f)
    range_sum_zero(g)
    range_sum(f, Nat.0) = Real.0
    range_sum(g, Nat.0) = Real.0
    Real.0 <= Real.0
    range_sum(f, Nat.0) <= range_sum(g, Nat.0)
    (forall(i: Nat) { i < Nat.0 implies f(i) <= g(i) }) implies range_sum(f, Nat.0) <= range_sum(g, Nat.0)
    p(Nat.0)
    forall(k: Nat) {
        if p(k) {
            if forall(i: Nat) { i < k.suc implies f(i) <= g(i) } {
                forall(i: Nat) {
                    if i < k {
                        lt_suc(k)
                        k < k.suc
                        lt_trans(i, k, k.suc)
                        i < k.suc
                        f(i) <= g(i)
                    }
                    (i < k implies f(i) <= g(i))
                }
                p(k) = ((forall(i: Nat) { i < k implies f(i) <= g(i) })
                    implies range_sum(f, k) <= range_sum(g, k))
                p(k)
                range_sum(f, k) <= range_sum(g, k)
                lt_suc(k)
                k < k.suc
                f(k) <= g(k)
                add_lte_add(range_sum(f, k), range_sum(g, k), f(k), g(k))
                range_sum(f, k) + f(k) <= range_sum(g, k) + g(k)
                range_sum_suc(f, k)
                range_sum_suc(g, k)
                range_sum(f, k.suc) <= range_sum(g, k.suc)
            }
            p(k.suc)
        }
        (p(k) implies p(k.suc))
    }
    p(Nat.0) and forall(k: Nat) {
        p(k) implies p(k.suc)
    }
    Nat.induction(p)
    p(n)
    if forall(i: Nat) { i < n implies f(i) <= g(i) } {
        p(n) = ((forall(i: Nat) { i < n implies f(i) <= g(i) })
            implies range_sum(f, n) <= range_sum(g, n))
        p(n)
        range_sum(f, n) <= range_sum(g, n)
    }
}

/// The lower theta summand: the indicator times `log 2`, a pointwise lower bound for the
/// theta weight.
define theta_lower_summand(i: Nat) -> Real {
    prime_indicator(i) * (from_nat[Real](Nat.2)).log.get_or_else(Real.0)
}

/// At a prime the theta summand is at least `log 2`; elsewhere both sides vanish.
theorem theta_weight_lower_bound(i: Nat) {
    theta_lower_summand(i) <= chebyshev_theta_weight(i)
} by {
    if i.is_prime {
        chebyshev_theta_weight(i) = if i.is_prime {
            (from_nat[Real](i)).log.get_or_else(Real.0)
        } else {
            Real.0
        }
        chebyshev_theta_weight(i) = (from_nat[Real](i)).log.get_or_else(Real.0)
        prime_indicator(i) = if i.is_prime { Real.1 } else { Real.0 }
        prime_indicator(i) = Real.1
        theta_lower_summand(i) = prime_indicator(i) * (from_nat[Real](Nat.2)).log.get_or_else(Real.0)
        theta_lower_summand(i) = Real.1 * (from_nat[Real](Nat.2)).log.get_or_else(Real.0)
        Real.1 * (from_nat[Real](Nat.2)).log.get_or_else(Real.0) = (from_nat[Real](Nat.2)).log.get_or_else(Real.0)
        theta_lower_summand(i) = (from_nat[Real](Nat.2)).log.get_or_else(Real.0)
        Nat.1 < i
        lt_imp_lte_suc(Nat.1, i)
        Nat.2 <= i
        Nat.2 != Nat.0
        log_value_monotone_nat(Nat.2, i)
        (from_nat[Real](Nat.2)).log.get_or_else(Real.0) <= (from_nat[Real](i)).log.get_or_else(Real.0)
        theta_lower_summand(i) <= chebyshev_theta_weight(i)
    }
    if not i.is_prime {
        chebyshev_theta_weight(i) = if i.is_prime {
            (from_nat[Real](i)).log.get_or_else(Real.0)
        } else {
            Real.0
        }
        chebyshev_theta_weight(i) = Real.0
        prime_indicator(i) = if i.is_prime { Real.1 } else { Real.0 }
        prime_indicator(i) = Real.0
        theta_lower_summand(i) = prime_indicator(i) * (from_nat[Real](Nat.2)).log.get_or_else(Real.0)
        theta_lower_summand(i) = Real.0 * (from_nat[Real](Nat.2)).log.get_or_else(Real.0)
        Real.0 * (from_nat[Real](Nat.2)).log.get_or_else(Real.0) = Real.0
        theta_lower_summand(i) = Real.0
        Real.0 <= Real.0
        theta_lower_summand(i) <= chebyshev_theta_weight(i)
    }
    theta_lower_summand(i) <= chebyshev_theta_weight(i)
}

/// The lower bound in the theta sum is the count times `log 2`.
theorem range_sum_theta_lower_summand(x: Nat) {
    range_sum(theta_lower_summand, x.suc) =
        prime_count(x) * (from_nat[Real](Nat.2)).log.get_or_else(Real.0)
} by {
    forall(i: Nat) {
        theta_lower_summand(i) = prime_indicator(i) * (from_nat[Real](Nat.2)).log.get_or_else(Real.0)
        mul_const_fn(prime_indicator, (from_nat[Real](Nat.2)).log.get_or_else(Real.0), i) =
            prime_indicator(i) * (from_nat[Real](Nat.2)).log.get_or_else(Real.0)
        theta_lower_summand(i) =
            mul_const_fn(prime_indicator, (from_nat[Real](Nat.2)).log.get_or_else(Real.0), i)
    }
    range_sum_congr(theta_lower_summand,
        mul_const_fn(prime_indicator, (from_nat[Real](Nat.2)).log.get_or_else(Real.0)), x.suc)
    range_sum(theta_lower_summand, x.suc) =
        range_sum(mul_const_fn(prime_indicator, (from_nat[Real](Nat.2)).log.get_or_else(Real.0)), x.suc)
    range_sum_mul_const_real(prime_indicator, (from_nat[Real](Nat.2)).log.get_or_else(Real.0), x.suc)
    range_sum(mul_const_fn(prime_indicator, (from_nat[Real](Nat.2)).log.get_or_else(Real.0)), x.suc) =
        range_sum(prime_indicator, x.suc) * (from_nat[Real](Nat.2)).log.get_or_else(Real.0)
    range_sum(theta_lower_summand, x.suc) =
        range_sum(prime_indicator, x.suc) * (from_nat[Real](Nat.2)).log.get_or_else(Real.0)
    prime_count(x) = range_sum(prime_indicator, x.suc)
    range_sum(theta_lower_summand, x.suc) =
        prime_count(x) * (from_nat[Real](Nat.2)).log.get_or_else(Real.0)
}

/// The crude lower bound on Chebyshev's theta function:
/// `pi(x) * log 2 <= theta(x)`.
///
/// Every prime at most `x` contributes `log p >= log 2`, and the composite positions
/// contribute nothing.
theorem prime_count_mul_log_two_lte_theta(x: Nat) {
    prime_count(x) * (from_nat[Real](Nat.2)).log.get_or_else(Real.0) <= chebyshev_theta(x)
} by {
    forall(i: Nat) {
        theta_weight_lower_bound(i)
        theta_lower_summand(i) <= chebyshev_theta_weight(i)
    }
    range_sum_lte_real(theta_lower_summand, chebyshev_theta_weight, x.suc)
    range_sum(theta_lower_summand, x.suc) <= range_sum(chebyshev_theta_weight, x.suc)
    range_sum_theta_lower_summand(x)
    range_sum(theta_lower_summand, x.suc) =
        prime_count(x) * (from_nat[Real](Nat.2)).log.get_or_else(Real.0)
    chebyshev_theta(x) = range_sum(chebyshev_theta_weight, x.suc)
    range_sum(chebyshev_theta_weight, x.suc) = chebyshev_theta(x)
    prime_count(x) * (from_nat[Real](Nat.2)).log.get_or_else(Real.0) <= range_sum(chebyshev_theta_weight, x.suc)
    lte_trans(prime_count(x) * (from_nat[Real](Nat.2)).log.get_or_else(Real.0),
        range_sum(chebyshev_theta_weight, x.suc), chebyshev_theta(x))
    prime_count(x) * (from_nat[Real](Nat.2)).log.get_or_else(Real.0) <= chebyshev_theta(x)
}

/// The upper theta summand at `x`: the indicator times `log x`.
define theta_upper_summand(x: Nat, i: Nat) -> Real {
    prime_indicator(i) * (from_nat[Real](x)).log.get_or_else(Real.0)
}

/// On the range below `x`, the theta summand is at most the indicator times `log x`.
theorem theta_weight_upper_bound(x: Nat, i: Nat) {
    i < x.suc implies chebyshev_theta_weight(i) <= theta_upper_summand(x, i)
} by {
    if i < x.suc {
        if i.is_prime {
            chebyshev_theta_weight(i) = if i.is_prime {
                (from_nat[Real](i)).log.get_or_else(Real.0)
            } else {
                Real.0
            }
            chebyshev_theta_weight(i) = (from_nat[Real](i)).log.get_or_else(Real.0)
            lt_suc_right(i, x)
            i = x or i < x
            if i = x {
                lte_ref(x)
                i <= x
            }
            if i < x {
                i <= x
            }
            i <= x
            Nat.1 < i
            i != Nat.0
            log_value_monotone_nat(i, x)
            (from_nat[Real](i)).log.get_or_else(Real.0) <= (from_nat[Real](x)).log.get_or_else(Real.0)
            prime_indicator(i) = if i.is_prime { Real.1 } else { Real.0 }
            prime_indicator(i) = Real.1
            theta_upper_summand(x, i) = prime_indicator(i) * (from_nat[Real](x)).log.get_or_else(Real.0)
            theta_upper_summand(x, i) = Real.1 * (from_nat[Real](x)).log.get_or_else(Real.0)
            Real.1 * (from_nat[Real](x)).log.get_or_else(Real.0) = (from_nat[Real](x)).log.get_or_else(Real.0)
            theta_upper_summand(x, i) = (from_nat[Real](x)).log.get_or_else(Real.0)
            chebyshev_theta_weight(i) <= theta_upper_summand(x, i)
        }
        if not i.is_prime {
            chebyshev_theta_weight(i) = if i.is_prime {
                (from_nat[Real](i)).log.get_or_else(Real.0)
            } else {
                Real.0
            }
            chebyshev_theta_weight(i) = Real.0
            prime_indicator(i) = if i.is_prime { Real.1 } else { Real.0 }
            prime_indicator(i) = Real.0
            theta_upper_summand(x, i) = prime_indicator(i) * (from_nat[Real](x)).log.get_or_else(Real.0)
            theta_upper_summand(x, i) = Real.0 * (from_nat[Real](x)).log.get_or_else(Real.0)
            Real.0 * (from_nat[Real](x)).log.get_or_else(Real.0) = Real.0
            theta_upper_summand(x, i) = Real.0
            Real.0 <= Real.0
            chebyshev_theta_weight(i) <= theta_upper_summand(x, i)
        }
        chebyshev_theta_weight(i) <= theta_upper_summand(x, i)
    }
}

/// The upper bound in the theta sum is the count times `log x`.
theorem range_sum_theta_upper_summand(x: Nat) {
    range_sum(theta_upper_summand(x), x.suc) =
        prime_count(x) * (from_nat[Real](x)).log.get_or_else(Real.0)
} by {
    forall(i: Nat) {
        theta_upper_summand(x, i) = prime_indicator(i) * (from_nat[Real](x)).log.get_or_else(Real.0)
        mul_const_fn(prime_indicator, (from_nat[Real](x)).log.get_or_else(Real.0), i) =
            prime_indicator(i) * (from_nat[Real](x)).log.get_or_else(Real.0)
        theta_upper_summand(x, i) =
            mul_const_fn(prime_indicator, (from_nat[Real](x)).log.get_or_else(Real.0), i)
    }
    range_sum_congr(theta_upper_summand(x),
        mul_const_fn(prime_indicator, (from_nat[Real](x)).log.get_or_else(Real.0)), x.suc)
    range_sum(theta_upper_summand(x), x.suc) =
        range_sum(mul_const_fn(prime_indicator, (from_nat[Real](x)).log.get_or_else(Real.0)), x.suc)
    range_sum_mul_const_real(prime_indicator, (from_nat[Real](x)).log.get_or_else(Real.0), x.suc)
    range_sum(mul_const_fn(prime_indicator, (from_nat[Real](x)).log.get_or_else(Real.0)), x.suc) =
        range_sum(prime_indicator, x.suc) * (from_nat[Real](x)).log.get_or_else(Real.0)
    range_sum(theta_upper_summand(x), x.suc) =
        range_sum(prime_indicator, x.suc) * (from_nat[Real](x)).log.get_or_else(Real.0)
    prime_count(x) = range_sum(prime_indicator, x.suc)
    range_sum(theta_upper_summand(x), x.suc) =
        prime_count(x) * (from_nat[Real](x)).log.get_or_else(Real.0)
}

/// The crude upper bound on Chebyshev's theta function:
/// `theta(x) <= pi(x) * log x`.
///
/// Every prime at most `x` contributes `log p <= log x`, and the composite positions
/// contribute nothing.  When `x` is below two both sides vanish, so the bound is stated for
/// all `x`.
theorem theta_lte_prime_count_mul_log(x: Nat) {
    chebyshev_theta(x) <= prime_count(x) * (from_nat[Real](x)).log.get_or_else(Real.0)
} by {
    forall(i: Nat) {
        theta_weight_upper_bound(x, i)
        i < x.suc implies chebyshev_theta_weight(i) <= theta_upper_summand(x, i)
    }
    range_sum_lte_real_on_range(chebyshev_theta_weight, theta_upper_summand(x), x.suc)
    range_sum(chebyshev_theta_weight, x.suc) <= range_sum(theta_upper_summand(x), x.suc)
    range_sum_theta_upper_summand(x)
    range_sum(theta_upper_summand(x), x.suc) =
        prime_count(x) * (from_nat[Real](x)).log.get_or_else(Real.0)
    chebyshev_theta(x) = range_sum(chebyshev_theta_weight, x.suc)
    range_sum(chebyshev_theta_weight, x.suc) = chebyshev_theta(x)
    range_sum(chebyshev_theta_weight, x.suc) <= prime_count(x) * (from_nat[Real](x)).log.get_or_else(Real.0)
    lte_trans(range_sum(chebyshev_theta_weight, x.suc),
        prime_count(x) * (from_nat[Real](x)).log.get_or_else(Real.0), chebyshev_theta(x))
    chebyshev_theta(x) <= prime_count(x) * (from_nat[Real](x)).log.get_or_else(Real.0)
}

// ---------------------------------------------------------------------------
// (c) The prime number theorem and its Chebyshev form
// ---------------------------------------------------------------------------

// The prime number theorem states `pi(x) ~ x / log x` as `x -> infinity`, i.e. the ratio
// `pi(x) * log x / x` tends to one.  It is equivalent to Chebyshev's form `psi(x) ~ x`:
// the forward direction `pi ~ x / log x` implies `psi ~ x` through the crude bounds of (b)
// and summation by parts, and the reverse direction needs the sharpened comparison
// `psi(x) - theta(x) = O(sqrt(x) log x)` (recorded, unproved, at the end of `chebyshev.ac`),
// which bounds the contribution of the prime powers with exponent at least two.  The whole
// equivalence is asymptotic analysis over the reals — limits, big-O, and asymptotic
// equivalence — which the library does not yet carry, so it is stated here for future work
// rather than proved.
// theorem prime_number_theorem_equiv_psi {
//     ...
// }

// ---------------------------------------------------------------------------
// (d) The Mertens function
// ---------------------------------------------------------------------------

// The Mertens function `M(x) = sum_{n <= x} mu(n)` is defined in `mobius_inversion.ac` as
// `mertens`, together with the small values `M(0) = 0` and `M(1) = 1`.  Its range-sum form,
// further small values, and the Möbius-inversion connection back to the primes are in
// `mertens_nt.ac`; the module graphs of `chebyshev.ac` and `mobius_inversion.ac` cannot be
// combined in one module (each defines the `Nat.congr_mod` attribute through
// `interface.ac` and `congruence.ac` respectively), so the Mertens content lives there.

// ---------------------------------------------------------------------------
// (e) Bertrand's postulate in counting form: pi(2n) - pi(n) >= 1
// ---------------------------------------------------------------------------

/// Two does not divide three.
theorem two_not_divides_three {
    not Nat.2.divides(Nat.3)
} by {
    if Nat.2.divides(Nat.3) {
        divides_self(Nat.2)
        Nat.2.divides(Nat.2)
        divides_sub(Nat.3, Nat.2, Nat.2)
        Nat.2.divides(Nat.3 - Nat.2)
        Nat.3 - Nat.2 = Nat.1
        Nat.2.divides(Nat.1)
        nat_two_prime
        Nat.2.is_prime
        prime_does_not_divide_one(Nat.2)
        not Nat.2.divides(Nat.1)
        false
    }
}

/// A natural strictly between one and three is two.
theorem between_one_and_three_is_two(k: Nat) {
    Nat.1 < k and k < Nat.3 implies k = Nat.2
} by {
    if Nat.1 < k and k < Nat.3 {
        lt_suc_right(k, Nat.2)
        k = Nat.2 or k < Nat.2
        if k < Nat.2 {
            lt_imp_lte_suc(Nat.1, k)
            Nat.2 <= k
            lte_and_lt(Nat.2, k, Nat.2)
            Nat.2 < Nat.2
            lt_not_ref(Nat.2)
            false
        } else {
            k = Nat.2
        }
        k = Nat.2
    }
}

/// Three is prime.
theorem nat_three_prime_local {
    Nat.3.is_prime
} by {
    Nat.1 < Nat.2
    lt_suc(Nat.2)
    Nat.2 < Nat.3
    lt_trans(Nat.1, Nat.2, Nat.3)
    Nat.1 < Nat.3
    forall(k: Nat) {
        if Nat.1 < k and k < Nat.3 {
            between_one_and_three_is_two(k)
            k = Nat.2
            two_not_divides_three
            not Nat.2.divides(Nat.3)
            not k.divides(Nat.3)
        }
        (Nat.1 < k and k < Nat.3 implies not k.divides(Nat.3))
    }
    no_proper_divisor_imp_prime(Nat.3)
    Nat.3.is_prime
}

/// A range sum of nonnegative terms is nonnegative.
theorem range_sum_nonneg(f: Nat -> Real, n: Nat) {
    (forall(i: Nat) { i < n implies Real.0 <= f(i) }) implies Real.0 <= range_sum(f, n)
} by {
    define p(x: Nat) -> Bool {
        (forall(i: Nat) { i < x implies Real.0 <= f(i) }) implies Real.0 <= range_sum(f, x)
    }
    range_sum_zero(f)
    range_sum(f, Nat.0) = Real.0
    Real.0 <= Real.0
    (forall(i: Nat) { i < Nat.0 implies Real.0 <= f(i) }) implies Real.0 <= range_sum(f, Nat.0)
    p(Nat.0)
    forall(k: Nat) {
        if p(k) {
            if forall(i: Nat) { i < k.suc implies Real.0 <= f(i) } {
                forall(i: Nat) {
                    if i < k {
                        lt_suc(k)
                        k < k.suc
                        lt_trans(i, k, k.suc)
                        i < k.suc
                        Real.0 <= f(i)
                    }
                    (i < k implies Real.0 <= f(i))
                }
                p(k) = ((forall(i: Nat) { i < k implies Real.0 <= f(i) })
                    implies Real.0 <= range_sum(f, k))
                p(k)
                Real.0 <= range_sum(f, k)
                lt_suc(k)
                k < k.suc
                Real.0 <= f(k)
                add_lte_add(Real.0, range_sum(f, k), Real.0, f(k))
                Real.0 + Real.0 <= range_sum(f, k) + f(k)
                Real.0 <= range_sum(f, k) + f(k)
                range_sum_suc(f, k)
                range_sum(f, k.suc) = range_sum(f, k) + f(k)
                Real.0 <= range_sum(f, k.suc)
            }
            p(k.suc)
        }
        (p(k) implies p(k.suc))
    }
    p(Nat.0) and forall(k: Nat) {
        p(k) implies p(k.suc)
    }
    Nat.induction(p)
    p(n)
}

/// A nonnegative range sum is at least any single term.
theorem range_sum_nonneg_term_ge(f: Nat -> Real, n: Nat, j: Nat) {
    (forall(i: Nat) { i < n implies Real.0 <= f(i) }) and j < n implies f(j) <= range_sum(f, n)
} by {
    if (forall(i: Nat) { i < n implies Real.0 <= f(i) }) and j < n {
        lt_imp_lte_suc(j, n)
        j.suc <= n
        add_sub(j.suc, n)
        (n - j.suc) + j.suc = n
        add_comm(n - j.suc, j.suc)
        j.suc + (n - j.suc) = n
        range_sum_split(f, j.suc, n - j.suc)
        range_sum(f, j.suc + (n - j.suc)) =
            range_sum(f, j.suc) + range_sum(shift_fn(f, j.suc), n - j.suc)
        range_sum(f, n) = range_sum(f, j.suc) + range_sum(shift_fn(f, j.suc), n - j.suc)
        range_sum_suc(f, j)
        range_sum(f, j.suc) = range_sum(f, j) + f(j)
        range_sum(f, n) = range_sum(f, j) + f(j) + range_sum(shift_fn(f, j.suc), n - j.suc)
        forall(i: Nat) {
            if i < j {
                lt_trans(i, j, n)
                i < n
                Real.0 <= f(i)
            }
            (i < j implies Real.0 <= f(i))
        }
        range_sum_nonneg(f, j)
        Real.0 <= range_sum(f, j)
        forall(i: Nat) {
            if i < n - j.suc {
                lt_add_left(j.suc, i, n - j.suc)
                j.suc + i < j.suc + (n - j.suc)
                j.suc + i < n
                Real.0 <= f(j.suc + i)
                shift_fn(f, j.suc, i) = f(j.suc + i)
                Real.0 <= shift_fn(f, j.suc, i)
            }
            (i < n - j.suc implies Real.0 <= shift_fn(f, j.suc, i))
        }
        range_sum_nonneg(shift_fn(f, j.suc), n - j.suc)
        Real.0 <= range_sum(shift_fn(f, j.suc), n - j.suc)
        add_lte_add(Real.0, range_sum(f, j), f(j), f(j))
        Real.0 + f(j) <= range_sum(f, j) + f(j)
        f(j) <= range_sum(f, j) + f(j)
        range_sum(f, j) + f(j) <= range_sum(f, j) + f(j)
        add_lte_add(range_sum(f, j) + f(j), range_sum(f, j) + f(j), Real.0, range_sum(shift_fn(f, j.suc), n - j.suc))
        (range_sum(f, j) + f(j)) + Real.0 <= (range_sum(f, j) + f(j)) + range_sum(shift_fn(f, j.suc), n - j.suc)
        range_sum(f, j) + f(j) <= range_sum(f, j) + f(j) + range_sum(shift_fn(f, j.suc), n - j.suc)
        lte_trans(f(j), range_sum(f, j) + f(j), range_sum(f, j) + f(j) + range_sum(shift_fn(f, j.suc), n - j.suc))
        f(j) <= range_sum(f, j) + f(j) + range_sum(shift_fn(f, j.suc), n - j.suc)
        f(j) <= range_sum(f, n)
    }
}

/// A prime of the form `n + 1 + j` with `j < n` forces the prime count to grow between
/// `n` and `2n`: `pi(2n) - pi(n) >= 1`.
///
/// The count difference is the sum of the prime indicator over the range `(n, 2n]`; the
/// prime `n + 1 + j` lies in that range and contributes exactly one, and all other terms
/// are nonnegative.
theorem prime_between_imp_count_gap(n: Nat, p: Nat, j: Nat) {
    p.is_prime and p = n.suc + j and j < n implies
        Real.1 <= prime_count(n + n) - prime_count(n)
} by {
    if p.is_prime and p = n.suc + j and j < n {
        prime_count(n + n) = range_sum(prime_indicator, (n + n).suc)
        (n + n).suc = n.suc + n
        range_sum_split(prime_indicator, n.suc, n)
        range_sum(prime_indicator, n.suc + n) =
            range_sum(prime_indicator, n.suc) + range_sum(shift_fn(prime_indicator, n.suc), n)
        range_sum(prime_indicator, (n + n).suc) =
            range_sum(prime_indicator, n.suc) + range_sum(shift_fn(prime_indicator, n.suc), n)
        prime_count(n) = range_sum(prime_indicator, n.suc)
        prime_count(n + n) = prime_count(n) + range_sum(shift_fn(prime_indicator, n.suc), n)
        prime_count(n + n) - prime_count(n) = range_sum(shift_fn(prime_indicator, n.suc), n)
        forall(i: Nat) {
            if i < n {
                prime_indicator_nonneg(n.suc + i)
                Real.0 <= prime_indicator(n.suc + i)
                shift_fn(prime_indicator, n.suc, i) = prime_indicator(n.suc + i)
                Real.0 <= shift_fn(prime_indicator, n.suc, i)
            }
            (i < n implies Real.0 <= shift_fn(prime_indicator, n.suc, i))
        }
        shift_fn(prime_indicator, n.suc, j) = prime_indicator(n.suc + j)
        prime_indicator(n.suc + j) = prime_indicator(p)
        prime_indicator_of_prime(p)
        prime_indicator(p) = Real.1
        shift_fn(prime_indicator, n.suc, j) = Real.1
        range_sum_nonneg_term_ge(shift_fn(prime_indicator, n.suc), n, j)
        shift_fn(prime_indicator, n.suc, j) <= range_sum(shift_fn(prime_indicator, n.suc), n)
        Real.1 <= range_sum(shift_fn(prime_indicator, n.suc), n)
        Real.1 <= prime_count(n + n) - prime_count(n)
    }
}

/// Bertrand's postulate in counting form at `n = 1`: `pi(2) - pi(1) >= 1`, witnessed by
/// the prime two.
theorem prime_count_gap_one {
    Real.1 <= prime_count(Nat.2) - prime_count(Nat.1)
} by {
    Nat.2 = Nat.1.suc + Nat.0
    Nat.0 < Nat.1
    nat_two_prime
    Nat.2.is_prime
    prime_between_imp_count_gap(Nat.1, Nat.2, Nat.0)
    Real.1 <= prime_count(Nat.1 + Nat.1) - prime_count(Nat.1)
    Nat.1 + Nat.1 = Nat.2
    Real.1 <= prime_count(Nat.2) - prime_count(Nat.1)
}

/// Bertrand's postulate in counting form at `n = 2`: `pi(4) - pi(2) >= 1`, witnessed by
/// the prime three.
theorem prime_count_gap_two {
    Real.1 <= prime_count(Nat.4) - prime_count(Nat.2)
} by {
    Nat.3 = Nat.2.suc + Nat.0
    Nat.0 < Nat.2
    nat_three_prime_local
    Nat.3.is_prime
    prime_between_imp_count_gap(Nat.2, Nat.3, Nat.0)
    Real.1 <= prime_count(Nat.2 + Nat.2) - prime_count(Nat.2)
    Nat.2 + Nat.2 = Nat.4
    Real.1 <= prime_count(Nat.4) - prime_count(Nat.2)
}

/// Bertrand's postulate in counting form at `n = 3`: `pi(6) - pi(3) >= 1`, witnessed by
/// the prime five.
theorem prime_count_gap_three {
    Real.1 <= prime_count(Nat.6) - prime_count(Nat.3)
} by {
    Nat.5 = Nat.3.suc + Nat.1
    Nat.1 < Nat.3
    five_is_prime
    Nat.5.is_prime
    prime_between_imp_count_gap(Nat.3, Nat.5, Nat.1)
    Real.1 <= prime_count(Nat.3 + Nat.3) - prime_count(Nat.3)
    Nat.3 + Nat.3 = Nat.6
    Real.1 <= prime_count(Nat.6) - prime_count(Nat.3)
}

/// Bertrand's postulate in counting form at `n = 4`: `pi(8) - pi(4) >= 1`, witnessed by
/// the prime five.
theorem prime_count_gap_four {
    Real.1 <= prime_count(Nat.8) - prime_count(Nat.4)
} by {
    Nat.5 = Nat.4.suc + Nat.0
    Nat.0 < Nat.4
    five_is_prime
    Nat.5.is_prime
    prime_between_imp_count_gap(Nat.4, Nat.5, Nat.0)
    Real.1 <= prime_count(Nat.4 + Nat.4) - prime_count(Nat.4)
    Nat.4 + Nat.4 = Nat.8
    Real.1 <= prime_count(Nat.8) - prime_count(Nat.4)
}

// The full statement — `pi(2n) - pi(n) >= 1` for every `n >= 1` — is exactly Bertrand's
// postulate in counting form: it is equivalent to the existence of a prime strictly between
// `n` and `2n` for every `n`.  That existence is the content of `bertrand_postulate` in
// `nat_bertrand.ac`, recorded there without proof: the elementary Erdős argument is
// assembled up to the analytic inequality, which is not yet in the library.  The bridge
// `prime_between_imp_count_gap` above turns any such prime into the counting inequality,
// and the four small cases above are proved directly.
// theorem bertrand_prime_count_gap(n: Nat) {
//     Nat.1 <= n implies Real.1 <= prime_count(n + n) - prime_count(n)
// }
