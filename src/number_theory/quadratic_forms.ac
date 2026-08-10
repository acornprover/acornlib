/// Binary quadratic forms: the two-square form x² + y², the difference of
/// squares x² - y², and the discriminant.
///
/// This file records the concrete algebra of binary quadratic forms:
///   - the Brahmagupta-Fibonacci identity (the product of two sums of two
///     squares is a sum of two squares), restated here from
///     `number_theory/sums_of_squares.ac`;
///   - the difference-of-squares identity over the reals;
///   - the discriminant b² - 4ac of the form ax² + bxy + cy², verified for
///     the form x² + y² (discriminant -4);
///   - the two-squares theorem, restated for primes congruent to one modulo
///     four;
///   - the product closure of the sums of two squares over the naturals,
///     proved from the Brahmagupta-Fibonacci identity.
///
/// The natural-number closure needs the case split of the
/// Brahmagupta-Fibonacci identity: natural subtraction is truncated, so the
/// product identity is stated with the hypothesis that decides which of the
/// two mixed products is the larger.
from nat import Nat
from nat import add_comm, add_assoc, add_comm_4, mul_comm, mul_assoc, distrib_left,
    distrib_right, lte_trans, lt_trans, lte_antisymm, lt_or_lte, lte_mul_both,
    add_sub, sub_lt, sub_self, sub_zero, add_imp_sub, add_imp_sub_left, sub_comm,
    lte_imp_not_lt, lt_imp_lte_suc, mul_suc_left, mul_suc_right, mul_two_left,
    lte_add_left, lte_ref, add_cancels_left, mul_to_zero
from order import lt_imp_lte
from real import Real
from number_theory.sums_of_squares import brahmagupta_fibonacci_identity,
    is_sum_two_squares, is_sum_two_squares_intro, is_sum_two_squares_apply
from number_theory.sum_of_two_squares import prime_sum_of_two_squares,
    prime_sum_two_squares_converse, lte_neq_imp_lt, not_lte_imp_gt

numerals Nat
numerals Real

// ============================================================================
// Section 1: the difference of squares over the reals
// ============================================================================

/// Opposite middle terms cancel in a sum of a difference and a negated
/// difference: a - c + (c - b) = a - b.
theorem qf_cancel_middle(a: Real, c: Real, b: Real) {
    a - c + (c - b) = a - b
} by {
    a - c + (c - b) = a + -c + (c + -b)
    a + -c + (c + -b) = a + (-c + (c + -b))
    -c + (c + -b) = (-c + c) + -b
    -c + c = Real.0
    (-c + c) + -b = Real.0 + -b
    Real.0 + -b = -b
    a + (-c + (c + -b)) = a + -b
    a + -b = a - b
}

/// The difference of squares: (x - y)(x + y) = x² - y².
theorem qf_square_diff(x: Real, y: Real) {
    (x - y) * (x + y) = x * x - y * y
} by {
    (x - y) * (x + y) = (x - y) * x + (x - y) * y
    (x - y) * x = x * x - y * x
    (x - y) * y = x * y - y * y
    (x - y) * (x + y) = x * x - y * x + (x * y - y * y)
    y * x = x * y
    (x - y) * (x + y) = x * x - x * y + (x * y - y * y)
    qf_cancel_middle(x * x, y * y, x * y)
    x * x - x * y + (x * y - y * y) = x * x - y * y
}

// ============================================================================
// Section 2: the Brahmagupta-Fibonacci identity over the reals
// ============================================================================

/// The Brahmagupta-Fibonacci identity: the product of two sums of two
/// squares is a sum of two squares,
///     (a² + b²)(c² + d²) = (ac - bd)² + (ad + bc)².
/// The identity is proved in `number_theory/sums_of_squares.ac`; it is
/// restated here so that the two-square forms are collected in this file.
theorem qf_brahmagupta_fibonacci(a: Real, b: Real, c: Real, d: Real) {
    (a * a + b * b) * (c * c + d * d) =
        (a * c - b * d) * (a * c - b * d) + (a * d + b * c) * (a * d + b * c)
} by {
    brahmagupta_fibonacci_identity(a, b, c, d)
    (a * a + b * b) * (c * c + d * d) =
        (a * c - b * d) * (a * c - b * d) + (a * d + b * c) * (a * d + b * c)
}

// ============================================================================
// Section 3: the discriminant
// ============================================================================

/// The real number two, 1 + 1 (the digit `Real.2` is not defined in the
/// library).
let real_two: Real = Real.1 + Real.1

/// The real number four, 2 + 2 (the digit `Real.4` is not defined in the
/// library).
let real_four: Real = real_two + real_two

/// The value a·x² + b·x·y + c·y² of the binary quadratic form
/// a·x² + b·x·y + c·y² at the point (x, y).
define quadratic_form_value(a: Real, b: Real, c: Real, x: Real, y: Real) -> Real {
    a * x * x + b * x * y + c * y * y
}

/// The discriminant b² - 4ac of the binary quadratic form
/// a·x² + b·x·y + c·y².
define quadratic_form_discriminant(a: Real, b: Real, c: Real) -> Real {
    b * b - real_four * a * c
}

/// The value of the form unfolds to the displayed expression.
theorem quadratic_form_value_unfold(a: Real, b: Real, c: Real, x: Real, y: Real) {
    quadratic_form_value(a, b, c, x, y) = a * x * x + b * x * y + c * y * y
} by {
}

/// The discriminant unfolds to b² - 4ac.
theorem quadratic_form_discriminant_unfold(a: Real, b: Real, c: Real) {
    quadratic_form_discriminant(a, b, c) = b * b - real_four * a * c
} by {
}

/// One times one is one.
theorem qf_one_mul_one {
    Real.1 * Real.1 = Real.1
} by {
}

/// Zero times zero is zero.
theorem qf_zero_mul_zero {
    Real.0 * Real.0 = Real.0
} by {
}

/// Four times one is four.
theorem qf_four_mul_one {
    real_four * Real.1 = real_four
} by {
}

/// Four times one times one is four.
theorem qf_four_mul_one_one {
    real_four * Real.1 * Real.1 = real_four
} by {
    real_four * Real.1 * Real.1 = real_four * (Real.1 * Real.1)
    Real.1 * Real.1 = Real.1
    real_four * (Real.1 * Real.1) = real_four * Real.1
    qf_four_mul_one
    real_four * Real.1 = real_four
    real_four * Real.1 * Real.1 = real_four
}

/// The discriminant of the form x² + y² (coefficients 1, 0, 1) is -4.
theorem quadratic_form_discriminant_x2_y2 {
    quadratic_form_discriminant(Real.1, Real.0, Real.1) = -(real_four)
} by {
    quadratic_form_discriminant(Real.1, Real.0, Real.1) =
        Real.0 * Real.0 - real_four * Real.1 * Real.1
    qf_zero_mul_zero
    Real.0 * Real.0 = Real.0
    qf_four_mul_one_one
    real_four * Real.1 * Real.1 = real_four
    Real.0 * Real.0 - real_four * Real.1 * Real.1 = Real.0 - real_four
    Real.0 - real_four = -(real_four)
    quadratic_form_discriminant(Real.1, Real.0, Real.1) = -(real_four)
}

// ============================================================================
// Section 4: the two-squares theorem
// ============================================================================

/// A prime congruent to one modulo four is a sum of two squares.  This is
/// Fermat's two-squares theorem for primes, proved in
/// `number_theory/sum_of_two_squares.ac`; it is restated here in terms of
/// the `is_sum_two_squares` predicate of `number_theory/sums_of_squares.ac`.
theorem prime_congr_one_mod_four_sum_two_squares(p: Nat) {
    p.is_prime and p.mod(Nat.4) = Nat.1 implies is_sum_two_squares(p)
} by {
    if p.is_prime and p.mod(Nat.4) = Nat.1 {
        prime_sum_of_two_squares(p)
        exists(a: Nat, b: Nat) { a * a + b * b = p }
        let (a: Nat, b: Nat) satisfy { a * a + b * b = p }
        a * a + b * b = p
        p = a * a + b * b
        is_sum_two_squares_intro(p, a, b)
        is_sum_two_squares(p)
    }
}

/// A prime that is a sum of two squares and is not two is congruent to one
/// modulo four.  This is the converse of the two-squares theorem for
/// primes, proved in `number_theory/sum_of_two_squares.ac`.
theorem prime_sum_two_squares_congr_one_mod_four(p: Nat) {
    p.is_prime and p != Nat.2 and is_sum_two_squares(p) implies p.mod(Nat.4) = Nat.1
} by {
    if p.is_prime and p != Nat.2 and is_sum_two_squares(p) {
        is_sum_two_squares_apply(p)
        exists(a: Nat, b: Nat) { p = a * a + b * b }
        let (a: Nat, b: Nat) satisfy { p = a * a + b * b }
        a * a + b * b = p
        prime_sum_two_squares_converse(p)
        p.mod(Nat.4) = Nat.1
    }
}

// The full two-squares characterisation — a positive natural number is a
// sum of two squares exactly when every prime congruent to three modulo
// four divides it to an even power — is recorded in
// `number_theory/sums_of_squares.ac`; the forward half (a prime congruent
// to one modulo four is a sum of two squares) is restated above.

// ============================================================================
// Section 5: the product closure of the sums of two squares
// ============================================================================
//
// The set of natural numbers that are sums of two squares is closed under
// multiplication.  Over the naturals the Brahmagupta-Fibonacci identity
// requires a case split because subtraction is truncated: if b·c <= a·d
// then
//     (a² + b²)(c² + d²) = (ac + bd)² + (ad - bc)²,
// and otherwise the same identity holds with bc - ad in place of ad - bc.
// The lemmas below expand squares, cancel the cross terms, and reassemble
// the two expansions.

/// A square of a sum expands over the naturals:
/// (x + y)² = x² + y·x + (x·y + y²).
theorem qf_sq_add(x: Nat, y: Nat) {
    (x + y) * (x + y) = x * x + y * x + (x * y + y * y)
} by {
    (x + y) * (x + y) = (x + y) * x + (x + y) * y
    (x + y) * x = x * x + y * x
    (x + y) * y = x * y + y * y
    (x + y) * (x + y) = x * x + y * x + (x * y + y * y)
}

/// Multiplication distributes over truncated subtraction on the left:
/// a·(b - c) = a·b - a·c.
theorem qf_sub_left_distrib(a: Nat, b: Nat, c: Nat) {
    a * (b - c) = a * b - a * c
} by {
    if c <= b {
        add_sub(b, c)
        b - c + c = b
        (b - c) + c = b
        a * ((b - c) + c) = a * b
        a * ((b - c) + c) = a * (b - c) + a * c
        a * (b - c) + a * c = a * b
        add_imp_sub_left(a * (b - c), a * c, a * b)
        a * b - a * c = a * (b - c)
        a * (b - c) = a * b - a * c
    } else {
        not_lte_imp_gt(c, b)
        b < c
        sub_lt(b, c)
        b - c = Nat.0
        a * (b - c) = a * Nat.0
        a * Nat.0 = Nat.0
        if a = Nat.0 {
            a * b = Nat.0
            a * c = Nat.0
            a * b - a * c = Nat.0 - Nat.0
            Nat.0 - Nat.0 = Nat.0
            a * b - a * c = Nat.0
        } else {
            a != Nat.0
            lt_imp_lte_suc(b, c)
            b.suc <= c
            lte_mul_both(a, b.suc, c)
            a * b.suc <= a * c
            mul_suc_right(a, b)
            a * b.suc = a * b + a
            a * b + a <= a * c
            if a = Nat.0 {
                false
            }
            Nat.1 <= a
            lte_add_left(Nat.1, a, a * b)
            a * b + Nat.1 <= a * b + a
            lte_trans(a * b + Nat.1, a * b + a, a * c)
            a * b + Nat.1 <= a * c
            a * b < a * c
            sub_lt(a * b, a * c)
            a * b - a * c = Nat.0
        }
        a * (b - c) = a * b - a * c
    }
}

/// Multiplication distributes over truncated subtraction on the right:
/// (a - b)·c = a·c - b·c.
theorem qf_sub_right_distrib(a: Nat, b: Nat, c: Nat) {
    (a - b) * c = a * c - b * c
} by {
    if b <= a {
        add_sub(a, b)
        a - b + b = a
        (a - b) + b = a
        ((a - b) + b) * c = a * c
        ((a - b) + b) * c = (a - b) * c + b * c
        (a - b) * c + b * c = a * c
        add_imp_sub_left((a - b) * c, b * c, a * c)
        a * c - b * c = (a - b) * c
        (a - b) * c = a * c - b * c
    } else {
        not_lte_imp_gt(b, a)
        a < b
        sub_lt(a, b)
        a - b = Nat.0
        (a - b) * c = Nat.0 * c
        Nat.0 * c = Nat.0
        (a - b) * c = Nat.0
        if c = Nat.0 {
            a * c = Nat.0
            b * c = Nat.0
            a * c - b * c = Nat.0 - Nat.0
            Nat.0 - Nat.0 = Nat.0
            a * c - b * c = Nat.0
        } else {
            c != Nat.0
            lt_imp_lte_suc(a, b)
            a.suc <= b
            lte_mul_both(c, a.suc, b)
            c * a.suc <= c * b
            mul_suc_left(c, a)
            c * a.suc = c * a + c
            c * a + c <= c * b
            if c = Nat.0 {
                false
            }
            Nat.1 <= c
            lte_add_left(Nat.1, c, c * a)
            c * a + Nat.1 <= c * a + c
            lte_trans(c * a + Nat.1, c * a + c, c * b)
            c * a + Nat.1 <= c * b
            c * a < c * b
            c * a = a * c
            c * b = b * c
            a * c < b * c
            sub_lt(a * c, b * c)
            a * c - b * c = Nat.0
        }
        (a - b) * c = a * c - b * c
    }
}

/// A square of a difference expands over the naturals, in the exact form
/// (x - y)² = (x² - x·y) - (x·y - y²) that keeps the truncation harmless.
theorem qf_sq_sub(x: Nat, y: Nat) {
    y <= x implies (x - y) * (x - y) = (x * x - x * y) - (x * y - y * y)
} by {
    if y <= x {
        qf_sub_right_distrib(x, y, x - y)
        (x - y) * (x - y) = x * (x - y) - y * (x - y)
        qf_sub_left_distrib(x, x, y)
        x * (x - y) = x * x - x * y
        qf_sub_left_distrib(y, x, y)
        y * (x - y) = y * x - y * y
        (x - y) * (x - y) = (x * x - x * y) - (y * x - y * y)
        y * x = x * y
        (x * x - x * y) - (y * x - y * y) = (x * x - x * y) - (x * y - y * y)
    }
}

/// The cross products of the two pairs commute: a·d·b·c = a·c·b·d.
theorem qf_mul_cross(a: Nat, b: Nat, c: Nat, d: Nat) {
    a * d * b * c = a * c * b * d
} by {
    a * d * b * c = a * (d * b) * c
    d * b = b * d
    a * (d * b) * c = a * (b * d) * c
    a * (b * d) * c = (a * b) * d * c
    (a * b) * d * c = (a * b) * (d * c)
    d * c = c * d
    (a * b) * (d * c) = (a * b) * (c * d)
    (a * b) * (c * d) = ((a * b) * c) * d
    ((a * b) * c) * d = (a * (b * c)) * d
    (a * (b * c)) * d = a * ((b * c) * d)
    a * ((b * c) * d) = a * (b * (c * d))
    a * c * b * d = a * ((c * b) * d)
    c * b = b * c
    a * ((c * b) * d) = a * ((b * c) * d)
    a * ((b * c) * d) = a * (b * (c * d))
    a * d * b * c = a * c * b * d
}

/// A square of a product regroups: b·c·b·c = b²·c².
theorem qf_sq_mul4(b: Nat, c: Nat) {
    b * c * b * c = b * b * c * c
} by {
    b * c * b * c = (b * c) * (b * c)
    (b * c) * (b * c) = b * (c * (b * c))
    c * (b * c) = (c * b) * c
    c * b = b * c
    c * (b * c) = (b * c) * c
    b * (c * (b * c)) = b * ((b * c) * c)
    b * ((b * c) * c) = (b * (b * c)) * c
    b * (b * c) = (b * b) * c
    (b * (b * c)) * c = ((b * b) * c) * c
    ((b * b) * c) * c = (b * b) * (c * c)
    b * c * b * c = b * b * c * c
}

/// A square of a product regroups, in the form a·c·a·c = a²·c².
theorem qf_sq_mul_gen(a: Nat, c: Nat) {
    a * c * a * c = a * a * c * c
} by {
    qf_sq_mul4(a, c)
    a * c * a * c = a * a * c * c
}

/// Twice the mixed product plus the bracket of the square of a difference
/// is the sum of the two squares:
/// x·y + x·y + ((x² - x·y) - (x·y - y²)) = x² + y².
theorem qf_sq_sum_cancel(x: Nat, y: Nat) {
    y <= x implies
    x * y + x * y + ((x * x - x * y) - (x * y - y * y)) = x * x + y * y
} by {
    if y <= x {
        // y*y <= x*y
        lte_mul_both(y, y, x)
        y * y <= y * x
        y * x = x * y
        y * y <= x * y
        // x*y <= x*x
        lte_mul_both(x, y, x)
        x * y <= x * x
        // x*y - y*y <= x*x - x*y
        qf_sub_left_distrib(y, x, y)
        y * (x - y) = y * x - y * y
        y * x = x * y
        y * (x - y) = x * y - y * y
        qf_sub_left_distrib(x, x, y)
        x * (x - y) = x * x - x * y
        lte_mul_both(x - y, y, x)
        y * (x - y) <= x * (x - y)
        x * y - y * y <= x * x - x * y
        // the chain: add back the subtracted pieces
        add_sub(x * x - x * y, x * y - y * y)
        (x * x - x * y) - (x * y - y * y) + (x * y - y * y) = x * x - x * y
        add_sub(x * y, y * y)
        x * y - y * y + y * y = x * y
        (x * x - x * y) - (x * y - y * y) + (x * y - y * y) + y * y =
            (x * x - x * y) + y * y
        (x * x - x * y) - (x * y - y * y) + ((x * y - y * y) + y * y) =
            (x * x - x * y) + y * y
        (x * x - x * y) - (x * y - y * y) + x * y = (x * x - x * y) + y * y
        (x * x - x * y) - (x * y - y * y) + x * y + x * y =
            (x * x - x * y) + y * y + x * y
        (x * x - x * y) - (x * y - y * y) + (x * y + x * y) =
            (x * x - x * y) + y * y + x * y
        (x * x - x * y) + y * y + x * y = (x * x - x * y) + (y * y + x * y)
        y * y + x * y = x * y + y * y
        (x * x - x * y) + (y * y + x * y) = (x * x - x * y) + (x * y + y * y)
        (x * x - x * y) + (x * y + y * y) = ((x * x - x * y) + x * y) + y * y
        add_sub(x * x, x * y)
        x * x - x * y + x * y = x * x
        ((x * x - x * y) + x * y) + y * y = x * x + y * y
        (x * x - x * y) + y * y + x * y = x * x + y * y
        (x * x - x * y) - (x * y - y * y) + (x * y + x * y) = x * x + y * y
        x * y + x * y + ((x * x - x * y) - (x * y - y * y)) = x * x + y * y
    }
}

/// The bracket of the square of a difference equals the sum of the squares
/// minus twice the mixed product:
/// (x² - x·y) - (x·y - y²) = x² + y² - (x·y + x·y).
theorem qf_sub_sub_sq(x: Nat, y: Nat) {
    y <= x implies
    (x * x - x * y) - (x * y - y * y) = x * x + y * y - (x * y + x * y)
} by {
    if y <= x {
        qf_sq_sum_cancel(x, y)
        x * y + x * y + ((x * x - x * y) - (x * y - y * y)) = x * x + y * y
        (x * x - x * y) - (x * y - y * y) + (x * y + x * y) = x * x + y * y
        add_imp_sub((x * x - x * y) - (x * y - y * y), x * y + x * y, x * x + y * y)
        x * x + y * y - (x * y + x * y) = (x * x - x * y) - (x * y - y * y)
        (x * x - x * y) - (x * y - y * y) = x * x + y * y - (x * y + x * y)
    }
}

/// Twice the mixed product plus the residual difference is the sum of the
/// two squares: x·y + x·y + (x² + y² - (x·y + x·y)) = x² + y².
theorem qf_sq_sum_cancel2(x: Nat, y: Nat) {
    y <= x implies
    x * y + x * y + (x * x + y * y - (x * y + x * y)) = x * x + y * y
} by {
    if y <= x {
        qf_sq_sum_cancel(x, y)
        x * y + x * y + ((x * x - x * y) - (x * y - y * y)) = x * x + y * y
        // zero is at most the bracket, so x*y + x*y <= x*x + y*y
        Nat.0 + ((x * x - x * y) - (x * y - y * y)) = (x * x - x * y) - (x * y - y * y)
        exists(c: Nat) { Nat.0 + c = (x * x - x * y) - (x * y - y * y) }
        Nat.0 <= (x * x - x * y) - (x * y - y * y)
        lte_add_left(x * y + x * y, Nat.0, (x * x - x * y) - (x * y - y * y))
        (x * y + x * y) + Nat.0 <= (x * y + x * y) + ((x * x - x * y) - (x * y - y * y))
        (x * y + x * y) + Nat.0 = x * y + x * y
        (x * y + x * y) + ((x * x - x * y) - (x * y - y * y)) = x * x + y * y
        x * y + x * y <= x * x + y * y
        add_sub(x * x + y * y, x * y + x * y)
        x * x + y * y - (x * y + x * y) + (x * y + x * y) = x * x + y * y
        x * y + x * y + (x * x + y * y - (x * y + x * y)) = x * x + y * y
    }
}

/// The left-hand side of the Brahmagupta-Fibonacci identity expands to the
/// sum of the four pure terms: (a² + b²)(c² + d²) = a²c² + a²d² + b²c² + b²d².
theorem qf_bf_lhs(a: Nat, b: Nat, c: Nat, d: Nat) {
    (a * a + b * b) * (c * c + d * d) =
        a * a * c * c + a * a * d * d + b * b * c * c + b * b * d * d
} by {
    (a * a + b * b) * (c * c + d * d) = (a * a + b * b) * (c * c) + (a * a + b * b) * (d * d)
    (a * a + b * b) * (c * c) = a * a * (c * c) + b * b * (c * c)
    (a * a + b * b) * (d * d) = a * a * (d * d) + b * b * (d * d)
    (a * a + b * b) * (c * c + d * d) =
        a * a * (c * c) + b * b * (c * c) + (a * a * (d * d) + b * b * (d * d))
    a * a * (c * c) = a * a * c * c
    b * b * (c * c) = b * b * c * c
    a * a * (d * d) = a * a * d * d
    b * b * (d * d) = b * b * d * d
    a * a * (c * c) + b * b * (c * c) + (a * a * (d * d) + b * b * (d * d)) =
        a * a * c * c + b * b * c * c + (a * a * d * d + b * b * d * d)
    (a * a * c * c + b * b * c * c) + (a * a * d * d + b * b * d * d) =
        (a * a * c * c + a * a * d * d) + (b * b * c * c + b * b * d * d)
    add_comm_4(a * a * c * c, b * b * c * c, a * a * d * d, b * b * d * d)
    (a * a * c * c + b * b * c * c) + (a * a * d * d + b * b * d * d) =
        (a * a * c * c + a * a * d * d) + (b * b * c * c + b * b * d * d)
    (a * a * c * c + a * a * d * d) + (b * b * c * c + b * b * d * d) =
        a * a * c * c + a * a * d * d + b * b * c * c + b * b * d * d
    (a * a + b * b) * (c * c + d * d) =
        a * a * c * c + a * a * d * d + b * b * c * c + b * b * d * d
}

/// If b·c <= a·d, then the square b²c² of the smaller mixed product is at
/// most the mixed product a·d·b·c.
theorem qf_mul_lte1(a: Nat, b: Nat, c: Nat, d: Nat) {
    b * c <= a * d implies b * c * b * c <= a * d * b * c
} by {
    if b * c <= a * d {
        lte_mul_both(b * c, b * c, a * d)
        (b * c) * (b * c) <= (b * c) * (a * d)
        (b * c) * (b * c) = b * c * b * c
        (b * c) * (a * d) = b * c * a * d
        b * c * a * d = a * d * b * c
        b * c * b * c <= a * d * b * c
    }
}

/// If b·c <= a·d, then the mixed product a·d·b·c is at most the square
/// a²d² of the larger mixed product.
theorem qf_mul_lte2(a: Nat, b: Nat, c: Nat, d: Nat) {
    b * c <= a * d implies a * d * b * c <= a * d * a * d
} by {
    if b * c <= a * d {
        lte_mul_both(a * d, b * c, a * d)
        (a * d) * (b * c) <= (a * d) * (a * d)
        (a * d) * (b * c) = a * d * b * c
        (a * d) * (a * d) = a * d * a * d
        a * d * b * c <= a * d * a * d
    }
}

/// The Brahmagupta-Fibonacci identity over the naturals: if b·c <= a·d,
/// then (a² + b²)(c² + d²) = (ac + bd)² + (ad - bc)².
///
/// With truncated natural subtraction the identity needs the hypothesis:
/// it is the difference ad - bc, not bc - ad, that is exact.
theorem qf_bf_nat(a: Nat, b: Nat, c: Nat, d: Nat) {
    b * c <= a * d implies
    (a * a + b * b) * (c * c + d * d) =
        (a * c + b * d) * (a * c + b * d) + (a * d - b * c) * (a * d - b * c)
} by {
    if b * c <= a * d {
        qf_sq_add(a * c, b * d)
        (a * c + b * d) * (a * c + b * d) =
            a * c * a * c + b * d * a * c + (a * c * b * d + b * d * b * d)
        qf_sq_sub(a * d, b * c)
        (a * d - b * c) * (a * d - b * c) =
            (a * d * a * d - a * d * b * c) - (a * d * b * c - b * c * b * c)
        qf_sq_mul_gen(a, c)
        a * c * a * c = a * a * c * c
        qf_sq_mul4(b, d)
        b * d * b * d = b * b * d * d
        qf_sq_mul4(a, d)
        a * d * a * d = a * a * d * d
        qf_sq_mul4(b, c)
        b * c * b * c = b * b * c * c
        qf_mul_cross(a, b, c, d)
        a * d * b * c = a * c * b * d
        b * d * a * c = a * c * b * d
        a * c * b * d = a * d * b * c
        a * d * b * c = a * c * b * d
        a * c * a * c + b * d * a * c + (a * c * b * d + b * d * b * d) +
            ((a * d * a * d - a * d * b * c) - (a * d * b * c - b * c * b * c)) =
            a * a * c * c + a * d * b * c + (a * d * b * c + b * b * d * d) +
            ((a * a * d * d - a * d * b * c) - (a * d * b * c - b * b * c * c))
        qf_sub_sub_sq(a * d, b * c)
        (a * d * a * d - a * d * b * c) - (a * d * b * c - b * c * b * c) =
            a * d * a * d + b * c * b * c - (a * d * b * c + a * d * b * c)
        a * d * a * d = a * a * d * d
        b * c * b * c = b * b * c * c
        a * d * a * d + b * c * b * c - (a * d * b * c + a * d * b * c) =
            a * a * d * d + b * b * c * c - (a * d * b * c + a * d * b * c)
        (a * d * a * d - a * d * b * c) - (a * d * b * c - b * c * b * c) =
            a * a * d * d + b * b * c * c - (a * d * b * c + a * d * b * c)
        a * a * c * c + a * d * b * c + (a * d * b * c + b * b * d * d) +
            ((a * a * d * d - a * d * b * c) - (a * d * b * c - b * b * c * c)) =
            a * a * c * c + a * d * b * c + (a * d * b * c + b * b * d * d) +
            (a * a * d * d + b * b * c * c - (a * d * b * c + a * d * b * c))
        // regroup the sum: p + m + (m + r) + x = p + r + (m + m + x)
        a * a * c * c + a * d * b * c + (a * d * b * c + b * b * d * d) +
            (a * a * d * d + b * b * c * c - (a * d * b * c + a * d * b * c)) =
            (a * a * c * c + a * d * b * c) + (a * d * b * c + b * b * d * d) +
            (a * a * d * d + b * b * c * c - (a * d * b * c + a * d * b * c))
        (a * a * c * c + a * d * b * c) + (a * d * b * c + b * b * d * d) +
            (a * a * d * d + b * b * c * c - (a * d * b * c + a * d * b * c)) =
            (a * a * c * c + a * d * b * c + a * d * b * c + b * b * d * d) +
            (a * a * d * d + b * b * c * c - (a * d * b * c + a * d * b * c))
        (a * a * c * c + a * d * b * c + a * d * b * c + b * b * d * d) +
            (a * a * d * d + b * b * c * c - (a * d * b * c + a * d * b * c)) =
            ((a * a * c * c + a * d * b * c + a * d * b * c) + b * b * d * d) +
            (a * a * d * d + b * b * c * c - (a * d * b * c + a * d * b * c))
        ((a * a * c * c + a * d * b * c + a * d * b * c) + b * b * d * d) +
            (a * a * d * d + b * b * c * c - (a * d * b * c + a * d * b * c)) =
            (a * a * c * c + (a * d * b * c + a * d * b * c)) + b * b * d * d +
            (a * a * d * d + b * b * c * c - (a * d * b * c + a * d * b * c))
        (a * a * c * c + (a * d * b * c + a * d * b * c)) + b * b * d * d +
            (a * a * d * d + b * b * c * c - (a * d * b * c + a * d * b * c)) =
            (a * a * c * c + (b * b * d * d + (a * d * b * c + a * d * b * c))) +
            (a * a * d * d + b * b * c * c - (a * d * b * c + a * d * b * c))
        (a * a * c * c + (b * b * d * d + (a * d * b * c + a * d * b * c))) +
            (a * a * d * d + b * b * c * c - (a * d * b * c + a * d * b * c)) =
            (a * a * c * c + b * b * d * d) + (a * d * b * c + a * d * b * c) +
            (a * a * d * d + b * b * c * c - (a * d * b * c + a * d * b * c))
        (a * a * c * c + b * b * d * d) + (a * d * b * c + a * d * b * c) +
            (a * a * d * d + b * b * c * c - (a * d * b * c + a * d * b * c)) =
            a * a * c * c + b * b * d * d + (a * d * b * c + a * d * b * c) +
            (a * a * d * d + b * b * c * c - (a * d * b * c + a * d * b * c))
        qf_sq_sum_cancel2(a * d, b * c)
        a * d * b * c + a * d * b * c +
            (a * d * a * d + b * c * b * c - (a * d * b * c + a * d * b * c)) =
            a * d * a * d + b * c * b * c
        a * d * a * d + b * c * b * c = a * a * d * d + b * b * c * c
        a * d * b * c + a * d * b * c +
            (a * a * d * d + b * b * c * c - (a * d * b * c + a * d * b * c)) =
            a * a * d * d + b * b * c * c
        a * a * c * c + b * b * d * d + (a * d * b * c + a * d * b * c) +
            (a * a * d * d + b * b * c * c - (a * d * b * c + a * d * b * c)) =
            a * a * c * c + b * b * d * d + (a * a * d * d + b * b * c * c)
        // final regroup: p + r + (s1 + s2) = p + s1 + s2 + r
        a * a * c * c + b * b * d * d + (a * a * d * d + b * b * c * c) =
            a * a * c * c + (b * b * d * d + (a * a * d * d + b * b * c * c))
        a * a * c * c + (b * b * d * d + (a * a * d * d + b * b * c * c)) =
            a * a * c * c + ((b * b * d * d + a * a * d * d) + b * b * c * c)
        a * a * c * c + ((b * b * d * d + a * a * d * d) + b * b * c * c) =
            (a * a * c * c + (b * b * d * d + a * a * d * d)) + b * b * c * c
        b * b * d * d + a * a * d * d = a * a * d * d + b * b * d * d
        (a * a * c * c + (b * b * d * d + a * a * d * d)) + b * b * c * c =
            (a * a * c * c + (a * a * d * d + b * b * d * d)) + b * b * c * c
        a * a * c * c + (a * a * d * d + b * b * d * d) =
            (a * a * c * c + a * a * d * d) + b * b * d * d
        (a * a * c * c + (a * a * d * d + b * b * d * d)) + b * b * c * c =
            ((a * a * c * c + a * a * d * d) + b * b * d * d) + b * b * c * c
        ((a * a * c * c + a * a * d * d) + b * b * d * d) + b * b * c * c =
            (a * a * c * c + a * a * d * d) + (b * b * d * d + b * b * c * c)
        b * b * d * d + b * b * c * c = b * b * c * c + b * b * d * d
        (a * a * c * c + a * a * d * d) + (b * b * d * d + b * b * c * c) =
            (a * a * c * c + a * a * d * d) + (b * b * c * c + b * b * d * d)
        (a * a * c * c + a * a * d * d) + (b * b * c * c + b * b * d * d) =
            a * a * c * c + a * a * d * d + b * b * c * c + b * b * d * d
        a * a * c * c + b * b * d * d + (a * a * d * d + b * b * c * c) =
            a * a * c * c + a * a * d * d + b * b * c * c + b * b * d * d
        a * a * c * c + a * d * b * c + (a * d * b * c + b * b * d * d) +
            ((a * a * d * d - a * d * b * c) - (a * d * b * c - b * b * c * c)) =
            a * a * c * c + a * a * d * d + b * b * c * c + b * b * d * d
        (a * c + b * d) * (a * c + b * d) + (a * d - b * c) * (a * d - b * c) =
            a * a * c * c + a * d * b * c + (a * d * b * c + b * b * d * d) +
            ((a * a * d * d - a * d * b * c) - (a * d * b * c - b * b * c * c))
        (a * c + b * d) * (a * c + b * d) + (a * d - b * c) * (a * d - b * c) =
            a * a * c * c + a * a * d * d + b * b * c * c + b * b * d * d
        qf_bf_lhs(a, b, c, d)
        (a * a + b * b) * (c * c + d * d) =
            a * a * c * c + a * a * d * d + b * b * c * c + b * b * d * d
        (a * a + b * b) * (c * c + d * d) =
            (a * c + b * d) * (a * c + b * d) + (a * d - b * c) * (a * d - b * c)
    }
}

/// The Brahmagupta-Fibonacci identity over the naturals, in the other
/// direction: if a·d <= b·c, then
/// (a² + b²)(c² + d²) = (ac + bd)² + (bc - ad)².
theorem qf_bf_nat_alt(a: Nat, b: Nat, c: Nat, d: Nat) {
    a * d <= b * c implies
    (a * a + b * b) * (c * c + d * d) =
        (a * c + b * d) * (a * c + b * d) + (b * c - a * d) * (b * c - a * d)
} by {
    if a * d <= b * c {
        d * a = a * d
        c * b = b * c
        d * a <= c * b
        qf_bf_nat(c, d, a, b)
        (c * c + d * d) * (a * a + b * b) =
            (c * a + d * b) * (c * a + d * b) + (c * b - d * a) * (c * b - d * a)
        (c * c + d * d) * (a * a + b * b) = (a * a + b * b) * (c * c + d * d)
        (a * a + b * b) * (c * c + d * d) =
            (c * a + d * b) * (c * a + d * b) + (c * b - d * a) * (c * b - d * a)
        c * a = a * c
        d * b = b * d
        (c * a + d * b) * (c * a + d * b) = (a * c + b * d) * (a * c + b * d)
        c * b - d * a = b * c - a * d
        (c * b - d * a) * (c * b - d * a) = (b * c - a * d) * (b * c - a * d)
        (a * a + b * b) * (c * c + d * d) =
            (a * c + b * d) * (a * c + b * d) + (b * c - a * d) * (b * c - a * d)
    }
}

/// The sums of two squares are closed under multiplication: if m and n are
/// sums of two squares, so is m·n.  The witnesses come from the
/// Brahmagupta-Fibonacci identity; the case split chooses which of the two
/// differences ad - bc and bc - ad is exact.
theorem sum_two_squares_closed_mul(m: Nat, n: Nat) {
    is_sum_two_squares(m) and is_sum_two_squares(n) implies is_sum_two_squares(m * n)
} by {
    if is_sum_two_squares(m) and is_sum_two_squares(n) {
        is_sum_two_squares_apply(m)
        exists(a: Nat, b: Nat) { m = a * a + b * b }
        let (a: Nat, b: Nat) satisfy { m = a * a + b * b }
        is_sum_two_squares_apply(n)
        exists(c: Nat, d: Nat) { n = c * c + d * d }
        let (c: Nat, d: Nat) satisfy { n = c * c + d * d }
        if b * c <= a * d {
            qf_bf_nat(a, b, c, d)
            (a * a + b * b) * (c * c + d * d) =
                (a * c + b * d) * (a * c + b * d) + (a * d - b * c) * (a * d - b * c)
            m * n = (a * c + b * d) * (a * c + b * d) + (a * d - b * c) * (a * d - b * c)
            is_sum_two_squares_intro(m * n, a * c + b * d, a * d - b * c)
            is_sum_two_squares(m * n)
        } else {
            not_lte_imp_gt(b * c, a * d)
            a * d < b * c
            lt_imp_lte(a * d, b * c)
            a * d <= b * c
            qf_bf_nat_alt(a, b, c, d)
            (a * a + b * b) * (c * c + d * d) =
                (a * c + b * d) * (a * c + b * d) + (b * c - a * d) * (b * c - a * d)
            m * n = (a * c + b * d) * (a * c + b * d) + (b * c - a * d) * (b * c - a * d)
            is_sum_two_squares_intro(m * n, a * c + b * d, b * c - a * d)
            is_sum_two_squares(m * n)
        }
    }
}
