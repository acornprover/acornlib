from nat import Nat
from nat import alt_induction, mul_to_zero, mul_cancel_left, divides_trans
from list import List, map, add_contains_left, add_contains_right, add_contains_or,
    sum, sum_add, map_add, map_contains, map_contains_of_contains,
    unique_list_sum, injective_map_is_unique, unique_same_contains_map_sum_eq
from pair import Pair
from data.basic.functions import is_injective_fn
from number_theory.dirichlet import divisor_quotient, divisor_quotient_cofactor,
    divisor_quotient_positive, divisor_of_positive_is_positive,
    dirichlet_term, dirichlet_term_apply, dirichlet_convolve,
    dirichlet_convolve_apply, dirichlet_convolve_at_zero
from number_theory.divisor_sum import divisor_list, divisor_list_contains_implies,
    divisor_list_contains_of, divisor_list_is_unique, divisors_up_to, divisors_up_to_member,
    divisors_up_to_zero, divisors_up_to_suc_yes, divisors_up_to_suc_no
numerals Nat

/// Nat specialization of the `sum` cons equation, used to keep later local
/// divisor-list inductions from timing out on large mapped terms.
theorem sum_cons_nat(head: Nat, tail: List[Nat]) {
    sum(List.cons(head, tail)) = head + sum(tail)
}

/// If `d | n` and `e | n/d`, then the product `d * e` is a divisor of `n`.
/// This is the basic arithmetic shape behind expanding nested divisor sums.
theorem divisor_pair_product_divides(n: Nat, d: Nat, e: Nat) {
    d.divides(n) and e.divides(divisor_quotient(n, d)) implies (d * e).divides(n)
} by {
    if d.divides(n) and e.divides(divisor_quotient(n, d)) {
        let q: Nat = divisor_quotient(n, d)
        divisor_quotient_cofactor(n, d)
        d * q = n
        let r: Nat satisfy { e * r = q }
        d * (e * r) = d * q
        d * (e * r) = n
        (d * e) * r = d * (e * r)
        (d * e) * r = n
        (d * e).divides(n)
    }
}

/// For a positive ambient value, taking the cofactor after multiplying nested
/// divisors reassociates as expected: `n / (d * e) = (n / d) / e`.
theorem divisor_quotient_pair_reassoc(n: Nat, d: Nat, e: Nat) {
    Nat.0 < n and d.divides(n) and e.divides(divisor_quotient(n, d)) implies
        divisor_quotient(n, d * e) = divisor_quotient(divisor_quotient(n, d), e)
} by {
    if Nat.0 < n and d.divides(n) and e.divides(divisor_quotient(n, d)) {
        let q: Nat = divisor_quotient(n, d)
        let r: Nat = divisor_quotient(q, e)
        divisor_quotient_cofactor(n, d)
        d * q = n
        divisor_quotient_cofactor(q, e)
        e * r = q
        d * (e * r) = d * q
        d * (e * r) = n
        (d * e) * r = d * (e * r)
        (d * e) * r = n
        divisor_pair_product_divides(n, d, e)
        (d * e).divides(n)
        divisor_quotient_cofactor(n, d * e)
        (d * e) * divisor_quotient(n, d * e) = n
        (d * e) * divisor_quotient(n, d * e) = (d * e) * r
        divisor_of_positive_is_positive(n, d)
        Nat.0 < d
        divisor_quotient_positive(n, d)
        Nat.0 < q
        divisor_of_positive_is_positive(q, e)
        Nat.0 < e
        if d * e = Nat.0 {
            mul_to_zero(d, e)
            d = Nat.0 or e = Nat.0
            false
        }
        d * e != Nat.0
        mul_cancel_left(d * e, divisor_quotient(n, d * e), r)
        divisor_quotient(n, d * e) = r
        r = divisor_quotient(q, e)
        q = divisor_quotient(n, d)
        divisor_quotient(n, d * e) = divisor_quotient(divisor_quotient(n, d), e)
    }
}

/// A nested divisor-list choice gives a divisor-list choice after multiplying
/// the two indices. This is a small list-membership bridge for later reindexing
/// of nested Dirichlet divisor sums.
theorem divisor_pair_product_in_divisor_list(n: Nat, d: Nat, e: Nat) {
    Nat.0 < n and divisor_list(n).contains(d) and
        divisor_list(divisor_quotient(n, d)).contains(e)
        implies divisor_list(n).contains(d * e)
} by {
    if Nat.0 < n and divisor_list(n).contains(d) and
            divisor_list(divisor_quotient(n, d)).contains(e) {
        divisor_list_contains_implies(n, d)
        Nat.0 < d and d.divides(n)
        Nat.0 < d
        d.divides(n)
        divisor_list_contains_implies(divisor_quotient(n, d), e)
        Nat.0 < e and e.divides(divisor_quotient(n, d))
        Nat.0 < e
        e.divides(divisor_quotient(n, d))
        divisor_pair_product_divides(n, d, e)
        (d * e).divides(n)
        if d * e = Nat.0 {
            mul_to_zero(d, e)
            d = Nat.0 or e = Nat.0
            false
        }
        d * e != Nat.0
        Nat.0 < d * e
        divisor_list_contains_of(n, d * e)
        divisor_list(n).contains(d * e)
    }
}

/// Pairs `(d, e)` with fixed outer divisor `d` and inner divisor
/// `e | divisor_quotient(n, d)`.  These are the natural indices for the
/// right-nested expansion of Dirichlet convolution at `n`.
define right_divisor_pair_block(n: Nat, d: Nat) -> List[Pair[Nat, Nat]] {
    map(divisor_list(divisor_quotient(n, d)), function(e: Nat) {
        Pair.new(d, e)
    })
}

/// Flatten the right-nested divisor-pair blocks over an explicit outer list.
define right_divisor_pair_list_from(n: Nat, outer: List[Nat]) -> List[Pair[Nat, Nat]] {
    match outer {
        List.nil {
            List.nil[Pair[Nat, Nat]]
        }
        List.cons(d, tail) {
            right_divisor_pair_block(n, d) + right_divisor_pair_list_from(n, tail)
        }
    }
}

/// The canonical right-nested divisor-pair list at `n`: all pairs `(d, e)` with
/// `d | n` and `e | divisor_quotient(n, d)`.
define right_divisor_pair_list(n: Nat) -> List[Pair[Nat, Nat]] {
    right_divisor_pair_list_from(n, divisor_list(n))
}

/// The term indexed by a canonical pair `(d, e)`, using the reassociated
/// remaining cofactor `divisor_quotient(n, d * e)`.
define divisor_pair_assoc_term(f: Nat -> Nat, g: Nat -> Nat, h: Nat -> Nat, n: Nat) ->
        (Pair[Nat, Nat] -> Nat) {
    function(p: Pair[Nat, Nat]) {
        f(p.first) * g(p.second) * h(divisor_quotient(n, p.first * p.second))
    }
}

/// Membership in a right divisor-pair block comes from the inner divisor list.
theorem right_divisor_pair_block_contains(n: Nat, d: Nat, e: Nat) {
    divisor_list(divisor_quotient(n, d)).contains(e)
        implies right_divisor_pair_block(n, d).contains(Pair.new(d, e))
} by {
    if divisor_list(divisor_quotient(n, d)).contains(e) {
        map_contains_of_contains(divisor_list(divisor_quotient(n, d)),
            function(x: Nat) { Pair.new(d, x) }, e)
        map(divisor_list(divisor_quotient(n, d)),
            function(x: Nat) { Pair.new(d, x) }).contains(Pair.new(d, e))
        right_divisor_pair_block(n, d).contains(Pair.new(d, e))
    }
}

/// Any member of a right divisor-pair block has the fixed first coordinate and
/// an inner divisor-list second coordinate.
theorem right_divisor_pair_block_contains_implies(n: Nat, d: Nat, p: Pair[Nat, Nat]) {
    right_divisor_pair_block(n, d).contains(p) implies
        p.first = d and divisor_list(divisor_quotient(n, d)).contains(p.second)
} by {
    if right_divisor_pair_block(n, d).contains(p) {
        right_divisor_pair_block(n, d) = map(divisor_list(divisor_quotient(n, d)),
            function(e: Nat) { Pair.new(d, e) })
        map_contains(divisor_list(divisor_quotient(n, d)),
            function(e: Nat) { Pair.new(d, e) }, p)
        let e: Nat satisfy {
            divisor_list(divisor_quotient(n, d)).contains(e) and Pair.new(d, e) = p
        }
        Pair.new(d, e).first = d
        Pair.new(d, e).second = e
        p.first = Pair.new(d, e).first
        p.second = Pair.new(d, e).second
        p.first = d
        p.second = e
        divisor_list(divisor_quotient(n, d)).contains(p.second)
        p.first = d and divisor_list(divisor_quotient(n, d)).contains(p.second)
    }
}

/// Membership in a right block is equivalent to the expected first coordinate
/// and inner-divisor membership.
theorem right_divisor_pair_block_contains_iff(n: Nat, d: Nat, p: Pair[Nat, Nat]) {
    right_divisor_pair_block(n, d).contains(p) =
        (p.first = d and divisor_list(divisor_quotient(n, d)).contains(p.second))
} by {
    if right_divisor_pair_block(n, d).contains(p) {
        right_divisor_pair_block_contains_implies(n, d, p)
        p.first = d and divisor_list(divisor_quotient(n, d)).contains(p.second)
    }
    if p.first = d and divisor_list(divisor_quotient(n, d)).contains(p.second) {
        right_divisor_pair_block_contains(n, d, p.second)
        right_divisor_pair_block(n, d).contains(Pair.new(d, p.second))
        Pair.new(p.first, p.second) = p
        Pair.new(d, p.second) = Pair.new(p.first, p.second)
        right_divisor_pair_block(n, d).contains(p)
    }
}

/// A pair whose first coordinate is the head of an explicit outer list and
/// whose second coordinate is an inner divisor belongs to the flattened right
/// list for that cons.
theorem right_divisor_pair_list_from_contains_head(n: Nat, head: Nat, tail: List[Nat],
        e: Nat) {
    divisor_list(divisor_quotient(n, head)).contains(e) implies
        right_divisor_pair_list_from(n, List.cons(head, tail)).contains(Pair.new(head, e))
} by {
    if divisor_list(divisor_quotient(n, head)).contains(e) {
        right_divisor_pair_block_contains(n, head, e)
        right_divisor_pair_block(n, head).contains(Pair.new(head, e))
        right_divisor_pair_list_from(n, List.cons(head, tail)) =
            right_divisor_pair_block(n, head) + right_divisor_pair_list_from(n, tail)
        add_contains_left(right_divisor_pair_block(n, head),
            right_divisor_pair_list_from(n, tail), Pair.new(head, e))
        right_divisor_pair_list_from(n, List.cons(head, tail)).contains(Pair.new(head, e))
    }
}

/// A pair that is already in the flattened tail also belongs to the flattened
/// right list after adding one outer head.
theorem right_divisor_pair_list_from_contains_tail(n: Nat, head: Nat, tail: List[Nat],
        p: Pair[Nat, Nat]) {
    right_divisor_pair_list_from(n, tail).contains(p) implies
        right_divisor_pair_list_from(n, List.cons(head, tail)).contains(p)
} by {
    if right_divisor_pair_list_from(n, tail).contains(p) {
        right_divisor_pair_list_from(n, List.cons(head, tail)) =
            right_divisor_pair_block(n, head) + right_divisor_pair_list_from(n, tail)
        add_contains_right(right_divisor_pair_block(n, head), right_divisor_pair_list_from(n, tail), p)
        right_divisor_pair_list_from(n, List.cons(head, tail)).contains(p)
    }
}

/// If a pair belongs to the head block of a flattened right list, its first
/// coordinate is the head and its second coordinate is an inner divisor.
theorem right_divisor_pair_list_from_head_contains_implies(n: Nat, head: Nat,
        tail: List[Nat], p: Pair[Nat, Nat]) {
    right_divisor_pair_block(n, head).contains(p) implies
        List.cons(head, tail).contains(p.first) and
            divisor_list(divisor_quotient(n, p.first)).contains(p.second)
} by {
    if right_divisor_pair_block(n, head).contains(p) {
        right_divisor_pair_block_contains_implies(n, head, p)
        p.first = head
        divisor_list(divisor_quotient(n, head)).contains(p.second)
        List.cons(head, tail).contains(head)
        List.cons(head, tail).contains(p.first)
        divisor_list(divisor_quotient(n, p.first)).contains(p.second)
        List.cons(head, tail).contains(p.first) and
            divisor_list(divisor_quotient(n, p.first)).contains(p.second)
    }
}

/// Bounded right-list membership introduction over `divisors_up_to`: if `d`
/// appears among the outer divisors up to `k` and `e` divides the cofactor of
/// `d`, then `(d, e)` appears in the flattened right pair list over that bound.
theorem right_divisor_pair_list_from_divisors_up_to_contains(n: Nat, k: Nat,
        d: Nat, e: Nat) {
    divisors_up_to(n, k).contains(d) and
        divisor_list(divisor_quotient(n, d)).contains(e) implies
        right_divisor_pair_list_from(n, divisors_up_to(n, k)).contains(Pair.new(d, e))
} by {
    define pred(bound: Nat) -> Bool {
        divisors_up_to(n, bound).contains(d) and
            divisor_list(divisor_quotient(n, d)).contains(e) implies
            right_divisor_pair_list_from(n, divisors_up_to(n, bound)).contains(Pair.new(d, e))
    }
    divisors_up_to_zero(n)
    divisors_up_to(n, Nat.0) = List.nil[Nat]
    right_divisor_pair_list_from(n, divisors_up_to(n, Nat.0)) = List.nil[Pair[Nat, Nat]]
    pred(Nat.0)
    forall(j: Nat) {
        if pred(j) {
            pred(j) = (divisors_up_to(n, j).contains(d) and
                divisor_list(divisor_quotient(n, d)).contains(e) implies
                right_divisor_pair_list_from(n, divisors_up_to(n, j)).contains(Pair.new(d, e)))
            if divisors_up_to(n, j.suc).contains(d) and
                    divisor_list(divisor_quotient(n, d)).contains(e) {
                if j.suc.divides(n) {
                    divisors_up_to_suc_yes(n, j)
                    divisors_up_to(n, j.suc) = List.cons(j.suc, divisors_up_to(n, j))
                    if d = j.suc {
                        right_divisor_pair_list_from_contains_head(n, j.suc,
                            divisors_up_to(n, j), e)
                        right_divisor_pair_list_from(n,
                            List.cons(j.suc, divisors_up_to(n, j))).contains(Pair.new(j.suc, e))
                        right_divisor_pair_list_from(n, divisors_up_to(n, j.suc)).contains(
                            Pair.new(d, e))
                    } else {
                        List.cons(j.suc, divisors_up_to(n, j)).contains(d)
                        if not divisors_up_to(n, j).contains(d) {
                            false
                        }
                        divisors_up_to(n, j).contains(d)
                        divisors_up_to(n, j).contains(d) and
                            divisor_list(divisor_quotient(n, d)).contains(e)
                        right_divisor_pair_list_from(n, divisors_up_to(n, j)).contains(
                            Pair.new(d, e))
                        right_divisor_pair_list_from_contains_tail(n, j.suc,
                            divisors_up_to(n, j), Pair.new(d, e))
                        right_divisor_pair_list_from(n,
                            List.cons(j.suc, divisors_up_to(n, j))).contains(Pair.new(d, e))
                        right_divisor_pair_list_from(n, divisors_up_to(n, j.suc)).contains(
                            Pair.new(d, e))
                    }
                } else {
                    not j.suc.divides(n)
                    divisors_up_to_suc_no(n, j)
                    divisors_up_to(n, j.suc) = divisors_up_to(n, j)
                    divisors_up_to(n, j).contains(d)
                    divisors_up_to(n, j).contains(d) and
                        divisor_list(divisor_quotient(n, d)).contains(e)
                    right_divisor_pair_list_from(n, divisors_up_to(n, j)).contains(Pair.new(d, e))
                    right_divisor_pair_list_from(n, divisors_up_to(n, j.suc)).contains(
                        Pair.new(d, e))
                }
            }
            pred(j.suc) = (divisors_up_to(n, j.suc).contains(d) and
                divisor_list(divisor_quotient(n, d)).contains(e) implies
                right_divisor_pair_list_from(n, divisors_up_to(n, j.suc)).contains(Pair.new(d, e)))
            pred(j.suc)
        }
    }
    forall(bound: Nat) { pred(bound) implies pred(bound.suc) }
    pred(Nat.0) and forall(bound: Nat) { pred(bound) implies pred(bound.suc) }
    alt_induction(pred)
    forall(bound: Nat) { pred(bound) }
    pred(k)
}

/// Canonical right-list membership introduction: if `d` is a positive divisor
/// of `n` in `divisor_list(n)` and `e` divides `n / d`, then `(d, e)` appears
/// in `right_divisor_pair_list(n)`.
theorem right_divisor_pair_list_contains(n: Nat, d: Nat, e: Nat) {
    divisor_list(n).contains(d) and divisor_list(divisor_quotient(n, d)).contains(e)
        implies right_divisor_pair_list(n).contains(Pair.new(d, e))
} by {
    if divisor_list(n).contains(d) and divisor_list(divisor_quotient(n, d)).contains(e) {
        right_divisor_pair_list(n) = right_divisor_pair_list_from(n, divisor_list(n))
        divisor_list(n) = divisors_up_to(n, n)
        right_divisor_pair_list_from_divisors_up_to_contains(n, n, d, e)
        right_divisor_pair_list_from(n, divisors_up_to(n, n)).contains(Pair.new(d, e))
        right_divisor_pair_list(n).contains(Pair.new(d, e))
    }
}

/// Bounded right-list membership elimination over `divisors_up_to`: every pair
/// in the flattened right pair list over a bound has first coordinate in that
/// outer divisor list and second coordinate in the corresponding cofactor list.
theorem right_divisor_pair_list_from_divisors_up_to_contains_implies(n: Nat, k: Nat,
        p: Pair[Nat, Nat]) {
    right_divisor_pair_list_from(n, divisors_up_to(n, k)).contains(p) implies
        divisors_up_to(n, k).contains(p.first) and
            divisor_list(divisor_quotient(n, p.first)).contains(p.second)
} by {
    define pred(bound: Nat) -> Bool {
        right_divisor_pair_list_from(n, divisors_up_to(n, bound)).contains(p) implies
            divisors_up_to(n, bound).contains(p.first) and
                divisor_list(divisor_quotient(n, p.first)).contains(p.second)
    }
    divisors_up_to_zero(n)
    divisors_up_to(n, Nat.0) = List.nil[Nat]
    right_divisor_pair_list_from(n, divisors_up_to(n, Nat.0)) = List.nil[Pair[Nat, Nat]]
    pred(Nat.0)
    forall(j: Nat) {
        if pred(j) {
            pred(j) = (right_divisor_pair_list_from(n, divisors_up_to(n, j)).contains(p) implies
                divisors_up_to(n, j).contains(p.first) and
                    divisor_list(divisor_quotient(n, p.first)).contains(p.second))
            if right_divisor_pair_list_from(n, divisors_up_to(n, j.suc)).contains(p) {
                if j.suc.divides(n) {
                    divisors_up_to_suc_yes(n, j)
                    divisors_up_to(n, j.suc) = List.cons(j.suc, divisors_up_to(n, j))
                    right_divisor_pair_list_from(n, divisors_up_to(n, j.suc)) =
                        right_divisor_pair_block(n, j.suc) +
                            right_divisor_pair_list_from(n, divisors_up_to(n, j))
                    add_contains_or(right_divisor_pair_block(n, j.suc),
                        right_divisor_pair_list_from(n, divisors_up_to(n, j)), p)
                    right_divisor_pair_block(n, j.suc).contains(p) or
                        right_divisor_pair_list_from(n, divisors_up_to(n, j)).contains(p)
                    if right_divisor_pair_block(n, j.suc).contains(p) {
                        right_divisor_pair_list_from_head_contains_implies(n, j.suc,
                            divisors_up_to(n, j), p)
                        List.cons(j.suc, divisors_up_to(n, j)).contains(p.first)
                        divisor_list(divisor_quotient(n, p.first)).contains(p.second)
                        divisors_up_to(n, j.suc).contains(p.first)
                        divisors_up_to(n, j.suc).contains(p.first) and
                            divisor_list(divisor_quotient(n, p.first)).contains(p.second)
                    } else {
                        right_divisor_pair_list_from(n, divisors_up_to(n, j)).contains(p)
                        divisors_up_to(n, j).contains(p.first) and
                            divisor_list(divisor_quotient(n, p.first)).contains(p.second)
                        divisors_up_to(n, j).contains(p.first)
                        divisor_list(divisor_quotient(n, p.first)).contains(p.second)
                        List.cons(j.suc, divisors_up_to(n, j)).contains(p.first)
                        divisors_up_to(n, j.suc).contains(p.first)
                        divisors_up_to(n, j.suc).contains(p.first) and
                            divisor_list(divisor_quotient(n, p.first)).contains(p.second)
                    }
                } else {
                    not j.suc.divides(n)
                    divisors_up_to_suc_no(n, j)
                    divisors_up_to(n, j.suc) = divisors_up_to(n, j)
                    right_divisor_pair_list_from(n, divisors_up_to(n, j)).contains(p)
                    divisors_up_to(n, j).contains(p.first) and
                        divisor_list(divisor_quotient(n, p.first)).contains(p.second)
                    divisors_up_to(n, j).contains(p.first)
                    divisor_list(divisor_quotient(n, p.first)).contains(p.second)
                    divisors_up_to(n, j.suc).contains(p.first)
                    divisors_up_to(n, j.suc).contains(p.first) and
                        divisor_list(divisor_quotient(n, p.first)).contains(p.second)
                }
                divisors_up_to(n, j.suc).contains(p.first) and
                    divisor_list(divisor_quotient(n, p.first)).contains(p.second)
            }
            pred(j.suc) = (right_divisor_pair_list_from(n, divisors_up_to(n, j.suc)).contains(p) implies
                divisors_up_to(n, j.suc).contains(p.first) and
                    divisor_list(divisor_quotient(n, p.first)).contains(p.second))
            pred(j.suc)
        }
    }
    forall(bound: Nat) { pred(bound) implies pred(bound.suc) }
    pred(Nat.0) and forall(bound: Nat) { pred(bound) implies pred(bound.suc) }
    alt_induction(pred)
    forall(bound: Nat) { pred(bound) }
    pred(k)
}

/// Canonical right-list membership elimination: every pair in
/// `right_divisor_pair_list(n)` has first coordinate in `divisor_list(n)` and
/// second coordinate in the divisor list of the corresponding cofactor.
theorem right_divisor_pair_list_contains_implies(n: Nat, p: Pair[Nat, Nat]) {
    right_divisor_pair_list(n).contains(p) implies
        divisor_list(n).contains(p.first) and
            divisor_list(divisor_quotient(n, p.first)).contains(p.second)
} by {
    if right_divisor_pair_list(n).contains(p) {
        right_divisor_pair_list(n) = right_divisor_pair_list_from(n, divisor_list(n))
        divisor_list(n) = divisors_up_to(n, n)
        right_divisor_pair_list_from_divisors_up_to_contains_implies(n, n, p)
        divisors_up_to(n, n).contains(p.first)
        divisor_list(divisor_quotient(n, p.first)).contains(p.second)
        divisor_list(n).contains(p.first)
        divisor_list(n).contains(p.first) and
            divisor_list(divisor_quotient(n, p.first)).contains(p.second)
    }
}

/// The canonical right divisor-pair list contains exactly the pairs `(d, e)`
/// with `d` in `divisor_list(n)` and `e` in the divisor list of `n / d`.
theorem right_divisor_pair_list_contains_iff(n: Nat, d: Nat, e: Nat) {
    right_divisor_pair_list(n).contains(Pair.new(d, e)) =
        (divisor_list(n).contains(d) and divisor_list(divisor_quotient(n, d)).contains(e))
} by {
    if right_divisor_pair_list(n).contains(Pair.new(d, e)) {
        right_divisor_pair_list_contains_implies(n, Pair.new(d, e))
        Pair.new(d, e).first = d
        Pair.new(d, e).second = e
        divisor_list(n).contains(d)
        divisor_list(divisor_quotient(n, d)).contains(e)
        divisor_list(n).contains(d) and divisor_list(divisor_quotient(n, d)).contains(e)
    }
    if divisor_list(n).contains(d) and divisor_list(divisor_quotient(n, d)).contains(e) {
        right_divisor_pair_list_contains(n, d, e)
        right_divisor_pair_list(n).contains(Pair.new(d, e))
    }
}

/// A block constructor over an explicit inner divisor list.  This keeps the
/// right-nested expansion proofs bounded over `divisors_up_to` rather than
/// introducing a generic flat-map API.
define right_divisor_pair_block_from(n: Nat, d: Nat, inner: List[Nat]) -> List[Pair[Nat, Nat]] {
    map(inner, function(e: Nat) {
        Pair.new(d, e)
    })
}

/// The right-nested inner summand before rewriting the cofactor into the
/// canonical pair term.
define right_nested_inner_term(f: Nat -> Nat, g: Nat -> Nat, h: Nat -> Nat, n: Nat,
        d: Nat) -> (Nat -> Nat) {
    function(e: Nat) {
        f(d) * dirichlet_term(g, h, divisor_quotient(n, d))(e)
    }
}

/// The accepted block definition is the explicit-inner block at the divisor
/// list of the cofactor.
theorem right_divisor_pair_block_eq_from(n: Nat, d: Nat) {
    right_divisor_pair_block(n, d) =
        right_divisor_pair_block_from(n, d, divisor_list(divisor_quotient(n, d)))
}

/// On genuine nested divisor choices, the scaled inner Dirichlet summand is the
/// canonical pair-indexed associativity summand.
theorem right_nested_inner_term_eq_pair_assoc_term(f: Nat -> Nat, g: Nat -> Nat,
        h: Nat -> Nat, n: Nat, d: Nat, e: Nat) {
    Nat.0 < n and d.divides(n) and e.divides(divisor_quotient(n, d)) implies
        right_nested_inner_term(f, g, h, n, d)(e) =
            divisor_pair_assoc_term(f, g, h, n)(Pair.new(d, e))
} by {
    if Nat.0 < n and d.divides(n) and e.divides(divisor_quotient(n, d)) {
        let q: Nat = divisor_quotient(n, d)
        dirichlet_term_apply(g, h, q, e)
        dirichlet_term(g, h, q)(e) = g(e) * h(divisor_quotient(q, e))
        right_nested_inner_term(f, g, h, n, d)(e) =
            f(d) * dirichlet_term(g, h, q)(e)
        right_nested_inner_term(f, g, h, n, d)(e) =
            f(d) * (g(e) * h(divisor_quotient(q, e)))
        divisor_quotient_pair_reassoc(n, d, e)
        divisor_quotient(n, d * e) = divisor_quotient(q, e)
        Pair.new(d, e).first = d
        Pair.new(d, e).second = e
        divisor_pair_assoc_term(f, g, h, n)(Pair.new(d, e)) =
            f(d) * g(e) * h(divisor_quotient(n, d * e))
        divisor_pair_assoc_term(f, g, h, n)(Pair.new(d, e)) =
            f(d) * g(e) * h(divisor_quotient(q, e))
        f(d) * (g(e) * h(divisor_quotient(q, e))) =
            f(d) * g(e) * h(divisor_quotient(q, e))
        right_nested_inner_term(f, g, h, n, d)(e) =
            divisor_pair_assoc_term(f, g, h, n)(Pair.new(d, e))
    }
}

/// Head term rewrite for the direct inner-sum expansion: the scaled inner
/// Dirichlet summand is the canonical pair-indexed summand.
theorem right_nested_inner_head_term_eq_pair_assoc_term(f: Nat -> Nat, g: Nat -> Nat,
        h: Nat -> Nat, n: Nat, d: Nat, e: Nat) {
    Nat.0 < n and d.divides(n) and e.divides(divisor_quotient(n, d)) implies
        f(d) * dirichlet_term(g, h, divisor_quotient(n, d))(e) =
            divisor_pair_assoc_term(f, g, h, n)(Pair.new(d, e))
} by {
    if Nat.0 < n and d.divides(n) and e.divides(divisor_quotient(n, d)) {
        right_nested_inner_term_eq_pair_assoc_term(f, g, h, n, d, e)
        right_nested_inner_term(f, g, h, n, d)(e) =
            divisor_pair_assoc_term(f, g, h, n)(Pair.new(d, e))
        right_nested_inner_term(f, g, h, n, d)(e) =
            f(d) * dirichlet_term(g, h, divisor_quotient(n, d))(e)
        f(d) * dirichlet_term(g, h, divisor_quotient(n, d))(e) =
            divisor_pair_assoc_term(f, g, h, n)(Pair.new(d, e))
    }
}

/// Bounded direct inner-sum expansion over `divisors_up_to`: for fixed
/// `d | n`, the scaled inner convolution sum over cofactor divisors up to `k`
/// is the sum over the corresponding explicit right-pair block.
theorem right_nested_inner_sum_divisors_up_to_eq_pair_block_from_sum(f: Nat -> Nat,
        g: Nat -> Nat, h: Nat -> Nat, n: Nat, d: Nat, k: Nat) {
    Nat.0 < n and d.divides(n) implies
        f(d) * sum(map(divisors_up_to(divisor_quotient(n, d), k),
            dirichlet_term(g, h, divisor_quotient(n, d)))) =
            sum(map(right_divisor_pair_block_from(n, d,
                    divisors_up_to(divisor_quotient(n, d), k)),
                divisor_pair_assoc_term(f, g, h, n)))
} by {
    let q: Nat = divisor_quotient(n, d)
    define pred(bound: Nat) -> Bool {
        Nat.0 < n and d.divides(n) implies
            f(d) * sum(map(divisors_up_to(q, bound), dirichlet_term(g, h, q))) =
                sum(map(right_divisor_pair_block_from(n, d, divisors_up_to(q, bound)),
                    divisor_pair_assoc_term(f, g, h, n)))
    }
    divisors_up_to_zero(q)
    divisors_up_to(q, Nat.0) = List.nil[Nat]
    map[Nat, Nat](List.nil[Nat], dirichlet_term(g, h, q)) = List.nil[Nat]
    sum[Nat](List.nil[Nat]) = Nat.0
    right_divisor_pair_block_from(n, d, List.nil[Nat]) = List.nil[Pair[Nat, Nat]]
    map[Pair[Nat, Nat], Nat](List.nil[Pair[Nat, Nat]], divisor_pair_assoc_term(f, g, h, n)) =
        List.nil[Nat]
    f(d) * Nat.0 = Nat.0
    pred(Nat.0)
    forall(j: Nat) {
        if pred(j) {
            pred(j) = (Nat.0 < n and d.divides(n) implies
                f(d) * sum(map(divisors_up_to(q, j), dirichlet_term(g, h, q))) =
                    sum(map(right_divisor_pair_block_from(n, d, divisors_up_to(q, j)),
                        divisor_pair_assoc_term(f, g, h, n))))
            if Nat.0 < n and d.divides(n) {
                if j.suc.divides(q) {
                    divisors_up_to_suc_yes(q, j)
                    divisors_up_to(q, j.suc) = List.cons(j.suc, divisors_up_to(q, j))
                    map(List.cons(j.suc, divisors_up_to(q, j)), dirichlet_term(g, h, q)) =
                        List.cons(dirichlet_term(g, h, q)(j.suc),
                            map(divisors_up_to(q, j), dirichlet_term(g, h, q)))
                    sum(map(divisors_up_to(q, j.suc), dirichlet_term(g, h, q))) =
                        sum(List.cons(dirichlet_term(g, h, q)(j.suc),
                            map(divisors_up_to(q, j), dirichlet_term(g, h, q))))
                    sum_cons_nat(dirichlet_term(g, h, q)(j.suc),
                        map(divisors_up_to(q, j), dirichlet_term(g, h, q)))
                    sum(List.cons(dirichlet_term(g, h, q)(j.suc),
                            map(divisors_up_to(q, j), dirichlet_term(g, h, q)))) =
                        dirichlet_term(g, h, q)(j.suc) +
                            sum(map(divisors_up_to(q, j), dirichlet_term(g, h, q)))
                    sum(map(divisors_up_to(q, j.suc), dirichlet_term(g, h, q))) =
                        dirichlet_term(g, h, q)(j.suc) +
                            sum(map(divisors_up_to(q, j), dirichlet_term(g, h, q)))
                    f(d) * sum(map(divisors_up_to(q, j.suc), dirichlet_term(g, h, q))) =
                        f(d) * (dirichlet_term(g, h, q)(j.suc) +
                            sum(map(divisors_up_to(q, j), dirichlet_term(g, h, q))))
                    f(d) * (dirichlet_term(g, h, q)(j.suc) +
                            sum(map(divisors_up_to(q, j), dirichlet_term(g, h, q)))) =
                        f(d) * dirichlet_term(g, h, q)(j.suc) +
                            f(d) * sum(map(divisors_up_to(q, j), dirichlet_term(g, h, q)))
                    pred(j)
                    f(d) * sum(map(divisors_up_to(q, j), dirichlet_term(g, h, q))) =
                        sum(map(right_divisor_pair_block_from(n, d, divisors_up_to(q, j)),
                            divisor_pair_assoc_term(f, g, h, n)))
                    map(List.cons(j.suc, divisors_up_to(q, j)),
                            function(e: Nat) { Pair.new(d, e) }) =
                        List.cons(Pair.new(d, j.suc),
                            map(divisors_up_to(q, j), function(e: Nat) { Pair.new(d, e) }))
                    right_divisor_pair_block_from(n, d, divisors_up_to(q, j)) =
                        map(divisors_up_to(q, j), function(e: Nat) { Pair.new(d, e) })
                    right_divisor_pair_block_from(n, d, divisors_up_to(q, j.suc)) =
                        List.cons(Pair.new(d, j.suc),
                            right_divisor_pair_block_from(n, d, divisors_up_to(q, j)))
                    map(right_divisor_pair_block_from(n, d, divisors_up_to(q, j.suc)),
                            divisor_pair_assoc_term(f, g, h, n)) =
                        List.cons(divisor_pair_assoc_term(f, g, h, n)(Pair.new(d, j.suc)),
                            map(right_divisor_pair_block_from(n, d, divisors_up_to(q, j)),
                                divisor_pair_assoc_term(f, g, h, n)))
                    sum(map(right_divisor_pair_block_from(n, d, divisors_up_to(q, j.suc)),
                            divisor_pair_assoc_term(f, g, h, n))) =
                        sum(List.cons(divisor_pair_assoc_term(f, g, h, n)(Pair.new(d, j.suc)),
                            map(right_divisor_pair_block_from(n, d, divisors_up_to(q, j)),
                                divisor_pair_assoc_term(f, g, h, n))))
                    sum_cons_nat(divisor_pair_assoc_term(f, g, h, n)(Pair.new(d, j.suc)),
                        map(right_divisor_pair_block_from(n, d, divisors_up_to(q, j)),
                            divisor_pair_assoc_term(f, g, h, n)))
                    sum(List.cons(divisor_pair_assoc_term(f, g, h, n)(Pair.new(d, j.suc)),
                            map(right_divisor_pair_block_from(n, d, divisors_up_to(q, j)),
                                divisor_pair_assoc_term(f, g, h, n)))) =
                        divisor_pair_assoc_term(f, g, h, n)(Pair.new(d, j.suc)) +
                            sum(map(right_divisor_pair_block_from(n, d, divisors_up_to(q, j)),
                                divisor_pair_assoc_term(f, g, h, n)))
                    sum(map(right_divisor_pair_block_from(n, d, divisors_up_to(q, j.suc)),
                            divisor_pair_assoc_term(f, g, h, n))) =
                        divisor_pair_assoc_term(f, g, h, n)(Pair.new(d, j.suc)) +
                            sum(map(right_divisor_pair_block_from(n, d, divisors_up_to(q, j)),
                                divisor_pair_assoc_term(f, g, h, n)))
                    right_nested_inner_head_term_eq_pair_assoc_term(f, g, h, n, d, j.suc)
                    f(d) * dirichlet_term(g, h, divisor_quotient(n, d))(j.suc) =
                        divisor_pair_assoc_term(f, g, h, n)(Pair.new(d, j.suc))
                    q = divisor_quotient(n, d)
                    dirichlet_term(g, h, q)(j.suc) =
                        dirichlet_term(g, h, divisor_quotient(n, d))(j.suc)
                    f(d) * dirichlet_term(g, h, q)(j.suc) =
                        divisor_pair_assoc_term(f, g, h, n)(Pair.new(d, j.suc))
                    f(d) * sum(map(divisors_up_to(q, j.suc), dirichlet_term(g, h, q))) =
                        sum(map(right_divisor_pair_block_from(n, d, divisors_up_to(q, j.suc)),
                            divisor_pair_assoc_term(f, g, h, n)))
                } else {
                    not j.suc.divides(q)
                    divisors_up_to_suc_no(q, j)
                    divisors_up_to(q, j.suc) = divisors_up_to(q, j)
                    pred(j)
                    f(d) * sum(map(divisors_up_to(q, j), dirichlet_term(g, h, q))) =
                        sum(map(right_divisor_pair_block_from(n, d, divisors_up_to(q, j)),
                            divisor_pair_assoc_term(f, g, h, n)))
                    f(d) * sum(map(divisors_up_to(q, j.suc), dirichlet_term(g, h, q))) =
                        sum(map(right_divisor_pair_block_from(n, d, divisors_up_to(q, j.suc)),
                            divisor_pair_assoc_term(f, g, h, n)))
                }
            }
            pred(j.suc) = (Nat.0 < n and d.divides(n) implies
                f(d) * sum(map(divisors_up_to(q, j.suc), dirichlet_term(g, h, q))) =
                    sum(map(right_divisor_pair_block_from(n, d, divisors_up_to(q, j.suc)),
                        divisor_pair_assoc_term(f, g, h, n))))
            pred(j.suc)
        }
    }
    forall(bound: Nat) { pred(bound) implies pred(bound.suc) }
    pred(Nat.0) and forall(bound: Nat) { pred(bound) implies pred(bound.suc) }
    alt_induction(pred)
    forall(bound: Nat) { pred(bound) }
    pred(k)
}

/// Sum form over the cofactor divisor list: a scaled inner convolution sum is
/// the sum over the corresponding block of canonical right-pair terms.
theorem right_nested_inner_sum_eq_pair_block_from_sum(f: Nat -> Nat, g: Nat -> Nat,
        h: Nat -> Nat, n: Nat, d: Nat) {
    Nat.0 < n and d.divides(n) implies
        f(d) * sum(map(divisor_list(divisor_quotient(n, d)),
            dirichlet_term(g, h, divisor_quotient(n, d)))) =
            sum(map(right_divisor_pair_block_from(n, d, divisor_list(divisor_quotient(n, d))),
                divisor_pair_assoc_term(f, g, h, n)))
} by {
    if Nat.0 < n and d.divides(n) {
        let q: Nat = divisor_quotient(n, d)
        q = divisor_quotient(n, d)
        right_nested_inner_sum_divisors_up_to_eq_pair_block_from_sum(f, g, h, n, d, q)
        f(d) * sum(map(divisors_up_to(divisor_quotient(n, d), q),
                dirichlet_term(g, h, divisor_quotient(n, d)))) =
            sum(map(right_divisor_pair_block_from(n, d, divisors_up_to(divisor_quotient(n, d), q)),
                divisor_pair_assoc_term(f, g, h, n)))
        dirichlet_term(g, h, divisor_quotient(n, d)) = dirichlet_term(g, h, q)
        divisors_up_to(divisor_quotient(n, d), q) = divisors_up_to(q, q)
        map(divisors_up_to(divisor_quotient(n, d), q),
                dirichlet_term(g, h, divisor_quotient(n, d))) =
            map(divisors_up_to(q, q), dirichlet_term(g, h, q))
        right_divisor_pair_block_from(n, d, divisors_up_to(divisor_quotient(n, d), q)) =
            right_divisor_pair_block_from(n, d, divisors_up_to(q, q))
        f(d) * sum(map(divisors_up_to(q, q), dirichlet_term(g, h, q))) =
            sum(map(right_divisor_pair_block_from(n, d, divisors_up_to(q, q)),
                divisor_pair_assoc_term(f, g, h, n)))
        divisor_list(q) = divisors_up_to(q, q)
        map(divisor_list(q), dirichlet_term(g, h, q)) =
            map(divisors_up_to(q, q), dirichlet_term(g, h, q))
        right_divisor_pair_block_from(n, d, divisor_list(q)) =
            right_divisor_pair_block_from(n, d, divisors_up_to(q, q))
        sum(map(right_divisor_pair_block_from(n, d, divisor_list(q)),
                divisor_pair_assoc_term(f, g, h, n))) =
            sum(map(right_divisor_pair_block_from(n, d, divisors_up_to(q, q)),
                divisor_pair_assoc_term(f, g, h, n)))
        f(d) * sum(map(divisor_list(q), dirichlet_term(g, h, q))) =
            sum(map(right_divisor_pair_block_from(n, d, divisor_list(q)),
                divisor_pair_assoc_term(f, g, h, n)))
        divisor_list(divisor_quotient(n, d)) = divisor_list(q)
        map(divisor_list(divisor_quotient(n, d)),
                dirichlet_term(g, h, divisor_quotient(n, d))) =
            map(divisor_list(q), dirichlet_term(g, h, q))
        right_divisor_pair_block_from(n, d, divisor_list(divisor_quotient(n, d))) =
            right_divisor_pair_block_from(n, d, divisor_list(q))
        f(d) * sum(map(divisor_list(divisor_quotient(n, d)),
            dirichlet_term(g, h, divisor_quotient(n, d)))) =
            sum(map(right_divisor_pair_block_from(n, d, divisor_list(divisor_quotient(n, d))),
                divisor_pair_assoc_term(f, g, h, n)))
    }
}

/// For a genuine outer divisor `d | n`, the `d`-summand of
/// `f * (g * h)` expands to the canonical sum over the right pair block for
/// that `d`.
theorem right_nested_outer_term_eq_pair_block_sum(f: Nat -> Nat, g: Nat -> Nat,
        h: Nat -> Nat, n: Nat, d: Nat) {
    Nat.0 < n and divisor_list(n).contains(d) implies
        dirichlet_term(f, dirichlet_convolve(g, h), n)(d) =
            sum(map(right_divisor_pair_block(n, d), divisor_pair_assoc_term(f, g, h, n)))
} by {
    if Nat.0 < n and divisor_list(n).contains(d) {
        divisor_list_contains_implies(n, d)
        Nat.0 < d and d.divides(n)
        d.divides(n)
        let q: Nat = divisor_quotient(n, d)
        dirichlet_term_apply(f, dirichlet_convolve(g, h), n, d)
        dirichlet_term(f, dirichlet_convolve(g, h), n)(d) =
            f(d) * dirichlet_convolve(g, h)(q)
        dirichlet_convolve_apply(g, h, q)
        dirichlet_convolve(g, h)(q) = sum(map(divisor_list(q), dirichlet_term(g, h, q)))
        dirichlet_term(f, dirichlet_convolve(g, h), n)(d) =
            f(d) * sum(map(divisor_list(q), dirichlet_term(g, h, q)))
        forall(e: Nat) {
            if divisor_list(q).contains(e) {
                divisor_list_contains_implies(q, e)
                Nat.0 < e and e.divides(q)
                e.divides(q)
                e.divides(divisor_quotient(n, d))
            }
        }
        right_nested_inner_sum_eq_pair_block_from_sum(f, g, h, n, d)
        f(d) * sum(map(divisor_list(q), dirichlet_term(g, h, q))) =
            sum(map(right_divisor_pair_block_from(n, d, divisor_list(q)),
                divisor_pair_assoc_term(f, g, h, n)))
        right_divisor_pair_block_eq_from(n, d)
        right_divisor_pair_block(n, d) = right_divisor_pair_block_from(n, d, divisor_list(q))
        sum(map(right_divisor_pair_block(n, d), divisor_pair_assoc_term(f, g, h, n))) =
            sum(map(right_divisor_pair_block_from(n, d, divisor_list(q)),
                divisor_pair_assoc_term(f, g, h, n)))
        dirichlet_term(f, dirichlet_convolve(g, h), n)(d) =
            sum(map(right_divisor_pair_block(n, d), divisor_pair_assoc_term(f, g, h, n)))
    }
}

/// Bounded outer expansion over `divisors_up_to`: the right-nested convolution
/// summands indexed by outer divisors up to `k` flatten to the accepted right
/// divisor-pair list over the same bound.
theorem right_nested_outer_sum_divisors_up_to_eq_pair_list_from_sum(f: Nat -> Nat,
        g: Nat -> Nat, h: Nat -> Nat, n: Nat, k: Nat) {
    Nat.0 < n implies
        sum(map(divisors_up_to(n, k), dirichlet_term(f, dirichlet_convolve(g, h), n))) =
            sum(map(right_divisor_pair_list_from(n, divisors_up_to(n, k)),
                divisor_pair_assoc_term(f, g, h, n)))
} by {
    define pred(bound: Nat) -> Bool {
        Nat.0 < n implies
            sum(map(divisors_up_to(n, bound), dirichlet_term(f, dirichlet_convolve(g, h), n))) =
                sum(map(right_divisor_pair_list_from(n, divisors_up_to(n, bound)),
                    divisor_pair_assoc_term(f, g, h, n)))
    }
    divisors_up_to_zero(n)
    divisors_up_to(n, Nat.0) = List.nil[Nat]
    map[Nat, Nat](List.nil[Nat], dirichlet_term(f, dirichlet_convolve(g, h), n)) = List.nil[Nat]
    sum[Nat](List.nil[Nat]) = Nat.0
    right_divisor_pair_list_from(n, List.nil[Nat]) = List.nil[Pair[Nat, Nat]]
    map[Pair[Nat, Nat], Nat](List.nil[Pair[Nat, Nat]], divisor_pair_assoc_term(f, g, h, n)) =
        List.nil[Nat]
    pred(Nat.0)
    forall(j: Nat) {
        if pred(j) {
            pred(j) = (Nat.0 < n implies
                sum(map(divisors_up_to(n, j), dirichlet_term(f, dirichlet_convolve(g, h), n))) =
                    sum(map(right_divisor_pair_list_from(n, divisors_up_to(n, j)),
                        divisor_pair_assoc_term(f, g, h, n))))
            if Nat.0 < n {
                if j.suc.divides(n) {
                    divisors_up_to_suc_yes(n, j)
                    divisors_up_to(n, j.suc) = List.cons(j.suc, divisors_up_to(n, j))
                    map(List.cons(j.suc, divisors_up_to(n, j)),
                            dirichlet_term(f, dirichlet_convolve(g, h), n)) =
                        List.cons(dirichlet_term(f, dirichlet_convolve(g, h), n)(j.suc),
                            map(divisors_up_to(n, j), dirichlet_term(f, dirichlet_convolve(g, h), n)))
                    sum(map(divisors_up_to(n, j.suc),
                            dirichlet_term(f, dirichlet_convolve(g, h), n))) =
                        sum(List.cons(dirichlet_term(f, dirichlet_convolve(g, h), n)(j.suc),
                            map(divisors_up_to(n, j), dirichlet_term(f, dirichlet_convolve(g, h), n))))
                    sum_cons_nat(dirichlet_term(f, dirichlet_convolve(g, h), n)(j.suc),
                        map(divisors_up_to(n, j), dirichlet_term(f, dirichlet_convolve(g, h), n)))
                    sum(map(divisors_up_to(n, j.suc),
                            dirichlet_term(f, dirichlet_convolve(g, h), n))) =
                        dirichlet_term(f, dirichlet_convolve(g, h), n)(j.suc) +
                            sum(map(divisors_up_to(n, j),
                                dirichlet_term(f, dirichlet_convolve(g, h), n)))
                    divisor_list_contains_of(n, j.suc)
                    divisor_list(n).contains(j.suc)
                    right_nested_outer_term_eq_pair_block_sum(f, g, h, n, j.suc)
                    dirichlet_term(f, dirichlet_convolve(g, h), n)(j.suc) =
                        sum(map(right_divisor_pair_block(n, j.suc),
                            divisor_pair_assoc_term(f, g, h, n)))
                    pred(j)
                    sum(map(divisors_up_to(n, j), dirichlet_term(f, dirichlet_convolve(g, h), n))) =
                        sum(map(right_divisor_pair_list_from(n, divisors_up_to(n, j)),
                            divisor_pair_assoc_term(f, g, h, n)))
                    right_divisor_pair_list_from(n, divisors_up_to(n, j.suc)) =
                        right_divisor_pair_block(n, j.suc) +
                            right_divisor_pair_list_from(n, divisors_up_to(n, j))
                    map_add(right_divisor_pair_block(n, j.suc),
                        right_divisor_pair_list_from(n, divisors_up_to(n, j)),
                        divisor_pair_assoc_term(f, g, h, n))
                    map(right_divisor_pair_block(n, j.suc) +
                            right_divisor_pair_list_from(n, divisors_up_to(n, j)),
                            divisor_pair_assoc_term(f, g, h, n)) =
                        map(right_divisor_pair_block(n, j.suc), divisor_pair_assoc_term(f, g, h, n)) +
                            map(right_divisor_pair_list_from(n, divisors_up_to(n, j)),
                                divisor_pair_assoc_term(f, g, h, n))
                    sum_add(map(right_divisor_pair_block(n, j.suc), divisor_pair_assoc_term(f, g, h, n)),
                        map(right_divisor_pair_list_from(n, divisors_up_to(n, j)),
                            divisor_pair_assoc_term(f, g, h, n)))
                    sum(map(right_divisor_pair_list_from(n, divisors_up_to(n, j.suc)),
                            divisor_pair_assoc_term(f, g, h, n))) =
                        sum(map(right_divisor_pair_block(n, j.suc), divisor_pair_assoc_term(f, g, h, n))) +
                            sum(map(right_divisor_pair_list_from(n, divisors_up_to(n, j)),
                                divisor_pair_assoc_term(f, g, h, n)))
                    sum(map(divisors_up_to(n, j.suc),
                            dirichlet_term(f, dirichlet_convolve(g, h), n))) =
                        sum(map(right_divisor_pair_list_from(n, divisors_up_to(n, j.suc)),
                            divisor_pair_assoc_term(f, g, h, n)))
                } else {
                    not j.suc.divides(n)
                    divisors_up_to_suc_no(n, j)
                    divisors_up_to(n, j.suc) = divisors_up_to(n, j)
                    pred(j)
                    sum(map(divisors_up_to(n, j), dirichlet_term(f, dirichlet_convolve(g, h), n))) =
                        sum(map(right_divisor_pair_list_from(n, divisors_up_to(n, j)),
                            divisor_pair_assoc_term(f, g, h, n)))
                    sum(map(divisors_up_to(n, j.suc), dirichlet_term(f, dirichlet_convolve(g, h), n))) =
                        sum(map(right_divisor_pair_list_from(n, divisors_up_to(n, j.suc)),
                            divisor_pair_assoc_term(f, g, h, n)))
                }
            }
            pred(j.suc) = (Nat.0 < n implies
                sum(map(divisors_up_to(n, j.suc), dirichlet_term(f, dirichlet_convolve(g, h), n))) =
                    sum(map(right_divisor_pair_list_from(n, divisors_up_to(n, j.suc)),
                        divisor_pair_assoc_term(f, g, h, n))))
            pred(j.suc)
        }
    }
    forall(bound: Nat) { pred(bound) implies pred(bound.suc) }
    pred(Nat.0) and forall(bound: Nat) { pred(bound) implies pred(bound.suc) }
    alt_induction(pred)
    forall(bound: Nat) { pred(bound) }
    pred(k)
}

/// One-sided fixed-positive-`n` expansion of the right-nested Dirichlet
/// convolution over the canonical right divisor-pair list.
theorem right_nested_convolve_eq_right_pair_list_sum(f: Nat -> Nat, g: Nat -> Nat,
        h: Nat -> Nat, n: Nat) {
    Nat.0 < n implies
        dirichlet_convolve(f, dirichlet_convolve(g, h))(n) =
            sum(map(right_divisor_pair_list(n), divisor_pair_assoc_term(f, g, h, n)))
} by {
    if Nat.0 < n {
        dirichlet_convolve_apply(f, dirichlet_convolve(g, h), n)
        dirichlet_convolve(f, dirichlet_convolve(g, h))(n) =
            sum(map(divisor_list(n), dirichlet_term(f, dirichlet_convolve(g, h), n)))
        right_nested_outer_sum_divisors_up_to_eq_pair_list_from_sum(f, g, h, n, n)
        divisor_list(n) = divisors_up_to(n, n)
        sum(map(divisor_list(n), dirichlet_term(f, dirichlet_convolve(g, h), n))) =
            sum(map(right_divisor_pair_list_from(n, divisor_list(n)),
                divisor_pair_assoc_term(f, g, h, n)))
        right_divisor_pair_list(n) = right_divisor_pair_list_from(n, divisor_list(n))
        sum(map(right_divisor_pair_list(n), divisor_pair_assoc_term(f, g, h, n))) =
            sum(map(right_divisor_pair_list_from(n, divisor_list(n)),
                divisor_pair_assoc_term(f, g, h, n)))
        dirichlet_convolve(f, dirichlet_convolve(g, h))(n) =
            sum(map(right_divisor_pair_list(n), divisor_pair_assoc_term(f, g, h, n)))
    }
}

/// The canonical pair generated by choosing an inner divisor `d` of a
/// left-nested outer divisor `r`.
define left_divisor_pair_of(r: Nat) -> (Nat -> Pair[Nat, Nat]) {
    function(d: Nat) {
        Pair.new(d, divisor_quotient(r, d))
    }
}

/// Pairs generated from a left-nested divisor choice: an outer divisor `r` and
/// an inner divisor `d | r`, encoded as the canonical pair `(d, r / d)`.
define left_divisor_pair_block(n: Nat, r: Nat) -> List[Pair[Nat, Nat]] {
    map(divisor_list(r), left_divisor_pair_of(r))
}

/// Flatten the left-nested divisor-pair blocks over an explicit outer list.
define left_divisor_pair_list_from(n: Nat, outer: List[Nat]) -> List[Pair[Nat, Nat]] {
    match outer {
        List.nil {
            List.nil[Pair[Nat, Nat]]
        }
        List.cons(r, tail) {
            left_divisor_pair_block(n, r) + left_divisor_pair_list_from(n, tail)
        }
    }
}

/// The canonical left-nested divisor-pair list at `n`: all pairs `(d, r / d)`
/// with `r | n` and `d | r`.
define left_divisor_pair_list(n: Nat) -> List[Pair[Nat, Nat]] {
    left_divisor_pair_list_from(n, divisor_list(n))
}

/// Membership introduction for a single left divisor-pair block.
theorem left_divisor_pair_block_contains(n: Nat, r: Nat, d: Nat) {
    divisor_list(r).contains(d) implies
        left_divisor_pair_block(n, r).contains(Pair.new(d, divisor_quotient(r, d)))
} by {
    if divisor_list(r).contains(d) {
        let make_pair = left_divisor_pair_of(r)
        make_pair(d) = Pair.new(d, divisor_quotient(r, d))
        map_contains_of_contains(divisor_list(r), make_pair, d)
        map(divisor_list(r), make_pair).contains(make_pair(d))
        map(divisor_list(r), make_pair).contains(Pair.new(d, divisor_quotient(r, d)))
        left_divisor_pair_block(n, r).contains(Pair.new(d, divisor_quotient(r, d)))
    }
}

/// A member of a left block has first coordinate in the inner divisor list and
/// second coordinate equal to the corresponding cofactor.
theorem left_divisor_pair_block_contains_implies(n: Nat, r: Nat, p: Pair[Nat, Nat]) {
    left_divisor_pair_block(n, r).contains(p) implies
        divisor_list(r).contains(p.first) and p.second = divisor_quotient(r, p.first)
} by {
    if left_divisor_pair_block(n, r).contains(p) {
        let make_pair = left_divisor_pair_of(r)
        left_divisor_pair_block(n, r) = map(divisor_list(r), make_pair)
        map_contains(divisor_list(r), make_pair, p)
        let d: Nat satisfy {
            divisor_list(r).contains(d) and make_pair(d) = p
        }
        make_pair(d) = Pair.new(d, divisor_quotient(r, d))
        Pair.new(d, divisor_quotient(r, d)) = p
        Pair.new(d, divisor_quotient(r, d)).first = d
        Pair.new(d, divisor_quotient(r, d)).second = divisor_quotient(r, d)
        p.first = Pair.new(d, divisor_quotient(r, d)).first
        p.second = Pair.new(d, divisor_quotient(r, d)).second
        p.first = d
        p.second = divisor_quotient(r, d)
        divisor_list(r).contains(p.first)
        p.second = divisor_quotient(r, p.first)
        divisor_list(r).contains(p.first) and p.second = divisor_quotient(r, p.first)
    }
}

/// First-coordinate projection from left-block membership.
theorem left_divisor_pair_block_contains_implies_first(n: Nat, r: Nat, p: Pair[Nat, Nat]) {
    left_divisor_pair_block(n, r).contains(p) implies divisor_list(r).contains(p.first)
} by {
    if left_divisor_pair_block(n, r).contains(p) {
        left_divisor_pair_block_contains_implies(n, r, p)
        divisor_list(r).contains(p.first) and p.second = divisor_quotient(r, p.first)
        divisor_list(r).contains(p.first)
    }
}

/// Second-coordinate projection from left-block membership.
theorem left_divisor_pair_block_contains_implies_second(n: Nat, r: Nat, p: Pair[Nat, Nat]) {
    left_divisor_pair_block(n, r).contains(p) implies p.second = divisor_quotient(r, p.first)
} by {
    if left_divisor_pair_block(n, r).contains(p) {
        left_divisor_pair_block_contains_implies(n, r, p)
        divisor_list(r).contains(p.first) and p.second = divisor_quotient(r, p.first)
        p.second = divisor_quotient(r, p.first)
    }
}

/// Canonical-pair block membership iff for the left block.
theorem left_divisor_pair_block_contains_iff(n: Nat, r: Nat, d: Nat, e: Nat) {
    left_divisor_pair_block(n, r).contains(Pair.new(d, e)) =
        (divisor_list(r).contains(d) and e = divisor_quotient(r, d))
} by {
    if left_divisor_pair_block(n, r).contains(Pair.new(d, e)) {
        left_divisor_pair_block_contains_implies(n, r, Pair.new(d, e))
        Pair.new(d, e).first = d
        Pair.new(d, e).second = e
        divisor_list(r).contains(d)
        e = divisor_quotient(r, d)
        divisor_list(r).contains(d) and e = divisor_quotient(r, d)
    }
    if divisor_list(r).contains(d) and e = divisor_quotient(r, d) {
        left_divisor_pair_block_contains(n, r, d)
        left_divisor_pair_block(n, r).contains(Pair.new(d, divisor_quotient(r, d)))
        Pair.new(d, e) = Pair.new(d, divisor_quotient(r, d))
        left_divisor_pair_block(n, r).contains(Pair.new(d, e))
    }
}

/// Product represented by a member of a left block: `(d, r / d)` multiplies
/// back to the outer divisor `r`.
theorem left_divisor_pair_block_contains_implies_product(n: Nat, r: Nat, p: Pair[Nat, Nat]) {
    left_divisor_pair_block(n, r).contains(p) implies p.first * p.second = r
} by {
    if left_divisor_pair_block(n, r).contains(p) {
        left_divisor_pair_block_contains_implies(n, r, p)
        divisor_list(r).contains(p.first)
        p.second = divisor_quotient(r, p.first)
        divisor_list_contains_implies(r, p.first)
        Nat.0 < p.first and p.first.divides(r)
        p.first.divides(r)
        divisor_quotient_cofactor(r, p.first)
        p.first * divisor_quotient(r, p.first) = r
        p.first * p.second = r
    }
}

/// A member of a left block also records that its first coordinate divides the
/// product of the pair.
theorem left_divisor_pair_block_contains_implies_inner_product(n: Nat, r: Nat,
        p: Pair[Nat, Nat]) {
    left_divisor_pair_block(n, r).contains(p) implies
        divisor_list(p.first * p.second).contains(p.first)
} by {
    if left_divisor_pair_block(n, r).contains(p) {
        left_divisor_pair_block_contains_implies_first(n, r, p)
        divisor_list(r).contains(p.first)
        left_divisor_pair_block_contains_implies_product(n, r, p)
        p.first * p.second = r
        divisor_list(p.first * p.second).contains(p.first)
    }
}

/// The second coordinate is the cofactor of the first in the product represented
/// by a member of a left block.
theorem left_divisor_pair_block_contains_implies_second_product(n: Nat, r: Nat,
        p: Pair[Nat, Nat]) {
    left_divisor_pair_block(n, r).contains(p) implies
        p.second = divisor_quotient(p.first * p.second, p.first)
} by {
    if left_divisor_pair_block(n, r).contains(p) {
        left_divisor_pair_block_contains_implies_second(n, r, p)
        p.second = divisor_quotient(r, p.first)
        left_divisor_pair_block_contains_implies_product(n, r, p)
        p.first * p.second = r
        p.second = divisor_quotient(p.first * p.second, p.first)
    }
}

/// A pair from the head block belongs to the flattened left list for that cons.
theorem left_divisor_pair_list_from_contains_head(n: Nat, head: Nat, tail: List[Nat],
        d: Nat) {
    divisor_list(head).contains(d) implies
        left_divisor_pair_list_from(n, List.cons(head, tail)).contains(
            Pair.new(d, divisor_quotient(head, d)))
} by {
    if divisor_list(head).contains(d) {
        left_divisor_pair_block_contains(n, head, d)
        left_divisor_pair_block(n, head).contains(Pair.new(d, divisor_quotient(head, d)))
        left_divisor_pair_list_from(n, List.cons(head, tail)) =
            left_divisor_pair_block(n, head) + left_divisor_pair_list_from(n, tail)
        add_contains_left(left_divisor_pair_block(n, head),
            left_divisor_pair_list_from(n, tail), Pair.new(d, divisor_quotient(head, d)))
        left_divisor_pair_list_from(n, List.cons(head, tail)).contains(
            Pair.new(d, divisor_quotient(head, d)))
    }
}

/// A pair already in the flattened tail remains present after adding a left
/// outer head.
theorem left_divisor_pair_list_from_contains_tail(n: Nat, head: Nat, tail: List[Nat],
        p: Pair[Nat, Nat]) {
    left_divisor_pair_list_from(n, tail).contains(p) implies
        left_divisor_pair_list_from(n, List.cons(head, tail)).contains(p)
} by {
    if left_divisor_pair_list_from(n, tail).contains(p) {
        left_divisor_pair_list_from(n, List.cons(head, tail)) =
            left_divisor_pair_block(n, head) + left_divisor_pair_list_from(n, tail)
        add_contains_right(left_divisor_pair_block(n, head),
            left_divisor_pair_list_from(n, tail), p)
        left_divisor_pair_list_from(n, List.cons(head, tail)).contains(p)
    }
}

/// If a pair belongs to the head block of a flattened left list, its product
/// is the head and hence belongs to the cons outer list.
theorem left_divisor_pair_list_from_head_contains_implies_product(n: Nat, head: Nat,
        tail: List[Nat], p: Pair[Nat, Nat]) {
    left_divisor_pair_block(n, head).contains(p) implies
        List.cons(head, tail).contains(p.first * p.second)
} by {
    if left_divisor_pair_block(n, head).contains(p) {
        left_divisor_pair_block_contains_implies_product(n, head, p)
        p.first * p.second = head
        List.cons(head, tail).contains(head)
        List.cons(head, tail).contains(p.first * p.second)
    }
}

/// Head-block projection: the first coordinate divides the represented product.
theorem left_divisor_pair_list_from_head_contains_implies_first_product(n: Nat,
        head: Nat, tail: List[Nat], p: Pair[Nat, Nat]) {
    left_divisor_pair_block(n, head).contains(p) implies
        divisor_list(p.first * p.second).contains(p.first)
} by {
    if left_divisor_pair_block(n, head).contains(p) {
        left_divisor_pair_block_contains_implies_inner_product(n, head, p)
        divisor_list(p.first * p.second).contains(p.first)
    }
}

/// Head-block projection: the second coordinate is the cofactor in the
/// represented product.
theorem left_divisor_pair_list_from_head_contains_implies_second_product(n: Nat,
        head: Nat, tail: List[Nat], p: Pair[Nat, Nat]) {
    left_divisor_pair_block(n, head).contains(p) implies
        p.second = divisor_quotient(p.first * p.second, p.first)
} by {
    if left_divisor_pair_block(n, head).contains(p) {
        left_divisor_pair_block_contains_implies_second_product(n, head, p)
        p.second = divisor_quotient(p.first * p.second, p.first)
    }
}

/// Bounded left-list membership introduction over `divisors_up_to`.
theorem left_divisor_pair_list_from_divisors_up_to_contains(n: Nat, k: Nat,
        r: Nat, d: Nat) {
    divisors_up_to(n, k).contains(r) and divisor_list(r).contains(d) implies
        left_divisor_pair_list_from(n, divisors_up_to(n, k)).contains(
            Pair.new(d, divisor_quotient(r, d)))
} by {
    define pred(bound: Nat) -> Bool {
        divisors_up_to(n, bound).contains(r) and divisor_list(r).contains(d) implies
            left_divisor_pair_list_from(n, divisors_up_to(n, bound)).contains(
                Pair.new(d, divisor_quotient(r, d)))
    }
    divisors_up_to_zero(n)
    divisors_up_to(n, Nat.0) = List.nil[Nat]
    left_divisor_pair_list_from(n, divisors_up_to(n, Nat.0)) = List.nil[Pair[Nat, Nat]]
    pred(Nat.0)
    forall(j: Nat) {
        if pred(j) {
            pred(j) = (divisors_up_to(n, j).contains(r) and divisor_list(r).contains(d) implies
                left_divisor_pair_list_from(n, divisors_up_to(n, j)).contains(
                    Pair.new(d, divisor_quotient(r, d))))
            if divisors_up_to(n, j.suc).contains(r) and divisor_list(r).contains(d) {
                if j.suc.divides(n) {
                    divisors_up_to_suc_yes(n, j)
                    divisors_up_to(n, j.suc) = List.cons(j.suc, divisors_up_to(n, j))
                    if r = j.suc {
                        left_divisor_pair_list_from_contains_head(n, j.suc, divisors_up_to(n, j), d)
                        left_divisor_pair_list_from(n,
                            List.cons(j.suc, divisors_up_to(n, j))).contains(
                                Pair.new(d, divisor_quotient(j.suc, d)))
                        left_divisor_pair_list_from(n, divisors_up_to(n, j.suc)).contains(
                            Pair.new(d, divisor_quotient(r, d)))
                    } else {
                        List.cons(j.suc, divisors_up_to(n, j)).contains(r)
                        if not divisors_up_to(n, j).contains(r) {
                            false
                        }
                        divisors_up_to(n, j).contains(r)
                        divisors_up_to(n, j).contains(r) and divisor_list(r).contains(d)
                        left_divisor_pair_list_from(n, divisors_up_to(n, j)).contains(
                            Pair.new(d, divisor_quotient(r, d)))
                        left_divisor_pair_list_from_contains_tail(n, j.suc, divisors_up_to(n, j),
                            Pair.new(d, divisor_quotient(r, d)))
                        left_divisor_pair_list_from(n,
                            List.cons(j.suc, divisors_up_to(n, j))).contains(
                                Pair.new(d, divisor_quotient(r, d)))
                        left_divisor_pair_list_from(n, divisors_up_to(n, j.suc)).contains(
                            Pair.new(d, divisor_quotient(r, d)))
                    }
                } else {
                    not j.suc.divides(n)
                    divisors_up_to_suc_no(n, j)
                    divisors_up_to(n, j.suc) = divisors_up_to(n, j)
                    divisors_up_to(n, j).contains(r)
                    divisors_up_to(n, j).contains(r) and divisor_list(r).contains(d)
                    left_divisor_pair_list_from(n, divisors_up_to(n, j)).contains(
                        Pair.new(d, divisor_quotient(r, d)))
                    left_divisor_pair_list_from(n, divisors_up_to(n, j.suc)).contains(
                        Pair.new(d, divisor_quotient(r, d)))
                }
            }
            pred(j.suc) = (divisors_up_to(n, j.suc).contains(r) and divisor_list(r).contains(d)
                implies left_divisor_pair_list_from(n, divisors_up_to(n, j.suc)).contains(
                    Pair.new(d, divisor_quotient(r, d))))
            pred(j.suc)
        }
    }
    forall(bound: Nat) { pred(bound) implies pred(bound.suc) }
    pred(Nat.0) and forall(bound: Nat) { pred(bound) implies pred(bound.suc) }
    alt_induction(pred)
    forall(bound: Nat) { pred(bound) }
    pred(k)
}

/// Canonical left-list membership introduction.
theorem left_divisor_pair_list_contains(n: Nat, r: Nat, d: Nat) {
    divisor_list(n).contains(r) and divisor_list(r).contains(d) implies
        left_divisor_pair_list(n).contains(Pair.new(d, divisor_quotient(r, d)))
} by {
    if divisor_list(n).contains(r) and divisor_list(r).contains(d) {
        left_divisor_pair_list(n) = left_divisor_pair_list_from(n, divisor_list(n))
        divisor_list(n) = divisors_up_to(n, n)
        left_divisor_pair_list_from_divisors_up_to_contains(n, n, r, d)
        left_divisor_pair_list_from(n, divisors_up_to(n, n)).contains(
            Pair.new(d, divisor_quotient(r, d)))
        left_divisor_pair_list(n).contains(Pair.new(d, divisor_quotient(r, d)))
    }
}

/// Bounded left-list membership elimination over `divisors_up_to`: a flattened
/// left-list pair has product in the outer list and first coordinate dividing
/// that product.
theorem left_divisor_pair_list_from_divisors_up_to_contains_implies(n: Nat, k: Nat,
        p: Pair[Nat, Nat]) {
    left_divisor_pair_list_from(n, divisors_up_to(n, k)).contains(p) implies
        divisors_up_to(n, k).contains(p.first * p.second) and
            divisor_list(p.first * p.second).contains(p.first) and
            p.second = divisor_quotient(p.first * p.second, p.first)
} by {
    define pred(bound: Nat) -> Bool {
        left_divisor_pair_list_from(n, divisors_up_to(n, bound)).contains(p) implies
            divisors_up_to(n, bound).contains(p.first * p.second) and
                divisor_list(p.first * p.second).contains(p.first) and
                p.second = divisor_quotient(p.first * p.second, p.first)
    }
    divisors_up_to_zero(n)
    divisors_up_to(n, Nat.0) = List.nil[Nat]
    left_divisor_pair_list_from(n, divisors_up_to(n, Nat.0)) = List.nil[Pair[Nat, Nat]]
    pred(Nat.0)
    forall(j: Nat) {
        if pred(j) {
            pred(j) = (left_divisor_pair_list_from(n, divisors_up_to(n, j)).contains(p) implies
                divisors_up_to(n, j).contains(p.first * p.second) and
                    divisor_list(p.first * p.second).contains(p.first) and
                    p.second = divisor_quotient(p.first * p.second, p.first))
            if left_divisor_pair_list_from(n, divisors_up_to(n, j.suc)).contains(p) {
                if j.suc.divides(n) {
                    divisors_up_to_suc_yes(n, j)
                    divisors_up_to(n, j.suc) = List.cons(j.suc, divisors_up_to(n, j))
                    left_divisor_pair_list_from(n, divisors_up_to(n, j.suc)) =
                        left_divisor_pair_block(n, j.suc) +
                            left_divisor_pair_list_from(n, divisors_up_to(n, j))
                    add_contains_or(left_divisor_pair_block(n, j.suc),
                        left_divisor_pair_list_from(n, divisors_up_to(n, j)), p)
                    left_divisor_pair_block(n, j.suc).contains(p) or
                        left_divisor_pair_list_from(n, divisors_up_to(n, j)).contains(p)
                    if left_divisor_pair_block(n, j.suc).contains(p) {
                        left_divisor_pair_list_from_head_contains_implies_product(n, j.suc,
                            divisors_up_to(n, j), p)
                        List.cons(j.suc, divisors_up_to(n, j)).contains(p.first * p.second)
                        left_divisor_pair_list_from_head_contains_implies_first_product(n, j.suc,
                            divisors_up_to(n, j), p)
                        divisor_list(p.first * p.second).contains(p.first)
                        left_divisor_pair_list_from_head_contains_implies_second_product(n, j.suc,
                            divisors_up_to(n, j), p)
                        p.second = divisor_quotient(p.first * p.second, p.first)
                        divisors_up_to(n, j.suc).contains(p.first * p.second)
                        divisors_up_to(n, j.suc).contains(p.first * p.second) and
                            divisor_list(p.first * p.second).contains(p.first) and
                            p.second = divisor_quotient(p.first * p.second, p.first)
                    } else {
                        left_divisor_pair_list_from(n, divisors_up_to(n, j)).contains(p)
                        divisors_up_to(n, j).contains(p.first * p.second) and
                            divisor_list(p.first * p.second).contains(p.first) and
                            p.second = divisor_quotient(p.first * p.second, p.first)
                        divisors_up_to(n, j).contains(p.first * p.second)
                        divisor_list(p.first * p.second).contains(p.first)
                        p.second = divisor_quotient(p.first * p.second, p.first)
                        List.cons(j.suc, divisors_up_to(n, j)).contains(p.first * p.second)
                        divisors_up_to(n, j.suc).contains(p.first * p.second)
                        divisors_up_to(n, j.suc).contains(p.first * p.second) and
                            divisor_list(p.first * p.second).contains(p.first) and
                            p.second = divisor_quotient(p.first * p.second, p.first)
                    }
                } else {
                    not j.suc.divides(n)
                    divisors_up_to_suc_no(n, j)
                    divisors_up_to(n, j.suc) = divisors_up_to(n, j)
                    left_divisor_pair_list_from(n, divisors_up_to(n, j)).contains(p)
                    divisors_up_to(n, j).contains(p.first * p.second) and
                        divisor_list(p.first * p.second).contains(p.first) and
                        p.second = divisor_quotient(p.first * p.second, p.first)
                    divisors_up_to(n, j).contains(p.first * p.second)
                    divisor_list(p.first * p.second).contains(p.first)
                    p.second = divisor_quotient(p.first * p.second, p.first)
                    divisors_up_to(n, j.suc).contains(p.first * p.second)
                    divisors_up_to(n, j.suc).contains(p.first * p.second) and
                        divisor_list(p.first * p.second).contains(p.first) and
                        p.second = divisor_quotient(p.first * p.second, p.first)
                }
                divisors_up_to(n, j.suc).contains(p.first * p.second) and
                    divisor_list(p.first * p.second).contains(p.first) and
                    p.second = divisor_quotient(p.first * p.second, p.first)
            }
            pred(j.suc) = (left_divisor_pair_list_from(n, divisors_up_to(n, j.suc)).contains(p)
                implies divisors_up_to(n, j.suc).contains(p.first * p.second) and
                    divisor_list(p.first * p.second).contains(p.first) and
                    p.second = divisor_quotient(p.first * p.second, p.first))
            pred(j.suc)
        }
    }
    forall(bound: Nat) { pred(bound) implies pred(bound.suc) }
    pred(Nat.0) and forall(bound: Nat) { pred(bound) implies pred(bound.suc) }
    alt_induction(pred)
    forall(bound: Nat) { pred(bound) }
    pred(k)
}

/// Canonical left-list membership elimination: a member has product in
/// `divisor_list(n)`, and its first coordinate divides that product.
theorem left_divisor_pair_list_contains_implies(n: Nat, p: Pair[Nat, Nat]) {
    left_divisor_pair_list(n).contains(p) implies
        divisor_list(n).contains(p.first * p.second) and
            divisor_list(p.first * p.second).contains(p.first) and
            p.second = divisor_quotient(p.first * p.second, p.first)
} by {
    if left_divisor_pair_list(n).contains(p) {
        left_divisor_pair_list(n) = left_divisor_pair_list_from(n, divisor_list(n))
        divisor_list(n) = divisors_up_to(n, n)
        left_divisor_pair_list_from(n, divisors_up_to(n, n)).contains(p)
        left_divisor_pair_list_from_divisors_up_to_contains_implies(n, n, p)
        divisors_up_to(n, n).contains(p.first * p.second) and
            divisor_list(p.first * p.second).contains(p.first) and
            p.second = divisor_quotient(p.first * p.second, p.first)
        divisors_up_to(n, n).contains(p.first * p.second)
        divisor_list(p.first * p.second).contains(p.first)
        p.second = divisor_quotient(p.first * p.second, p.first)
        divisor_list(n).contains(p.first * p.second)
        divisor_list(n).contains(p.first * p.second) and
            divisor_list(p.first * p.second).contains(p.first) and
            p.second = divisor_quotient(p.first * p.second, p.first)
    }
}

/// Membership in the canonical left divisor-pair list can be rewritten as the
/// right-list membership condition: first choose the represented product `d*e`
/// as the left-nested outer divisor, then choose `d` inside it.
theorem left_pair_membership_implies_right_membership(n: Nat, p: Pair[Nat, Nat]) {
    left_divisor_pair_list(n).contains(p) implies right_divisor_pair_list(n).contains(p)
} by {
    if left_divisor_pair_list(n).contains(p) {
        left_divisor_pair_list_contains_implies(n, p)
        divisor_list(n).contains(p.first * p.second)
        divisor_list(p.first * p.second).contains(p.first)
        p.second = divisor_quotient(p.first * p.second, p.first)

        divisor_list(n) = divisors_up_to(n, n)
        divisors_up_to(n, n).contains(p.first * p.second)
        divisors_up_to_member(n, n, p.first * p.second)
        Nat.0 < p.first * p.second and p.first * p.second <= n and
            (p.first * p.second).divides(n)
        Nat.0 < p.first * p.second
        Nat.0 < n
        (p.first * p.second).divides(n)
        divisor_list_contains_implies(p.first * p.second, p.first)
        Nat.0 < p.first and p.first.divides(p.first * p.second)
        p.first.divides(p.first * p.second)
        divides_trans(p.first, p.first * p.second, n)
        p.first.divides(n)
        divisor_quotient_cofactor(p.first * p.second, p.first)
        p.first * divisor_quotient(p.first * p.second, p.first) = p.first * p.second
        p.first * p.second = p.first * divisor_quotient(p.first * p.second, p.first)
        Nat.0 < p.first
        p.first != Nat.0
        mul_cancel_left(p.first, p.second, divisor_quotient(p.first * p.second, p.first))
        p.second = divisor_quotient(p.first * p.second, p.first)

        divisor_quotient_cofactor(n, p.first * p.second)
        (p.first * p.second) * divisor_quotient(n, p.first * p.second) = n
        p.first * (p.second * divisor_quotient(n, p.first * p.second)) =
            (p.first * p.second) * divisor_quotient(n, p.first * p.second)
        p.first * (p.second * divisor_quotient(n, p.first * p.second)) = n
        divisor_quotient_cofactor(n, p.first)
        p.first * divisor_quotient(n, p.first) = n
        p.first * divisor_quotient(n, p.first) =
            p.first * (p.second * divisor_quotient(n, p.first * p.second))
        mul_cancel_left(p.first, divisor_quotient(n, p.first),
            p.second * divisor_quotient(n, p.first * p.second))
        divisor_quotient(n, p.first) = p.second * divisor_quotient(n, p.first * p.second)
        p.second.divides(divisor_quotient(n, p.first))
        if p.second = Nat.0 {
            p.first * p.second = Nat.0
            false
        }
        p.second != Nat.0
        Nat.0 < p.second
        divisor_quotient_positive(n, p.first)
        Nat.0 < divisor_quotient(n, p.first)
        divisor_list_contains_of(divisor_quotient(n, p.first), p.second)
        divisor_list(divisor_quotient(n, p.first)).contains(p.second)
        divisor_list_contains_of(n, p.first)
        divisor_list(n).contains(p.first)
        right_divisor_pair_list_contains(n, p.first, p.second)
        right_divisor_pair_list(n).contains(Pair.new(p.first, p.second))
        Pair.new(p.first, p.second) = p
        right_divisor_pair_list(n).contains(p)
    }
}

/// Membership in the canonical right divisor-pair list can be rewritten as the
/// left-list membership condition by taking the product `d*e` as the represented
/// outer divisor on the left side.
theorem right_pair_membership_implies_left_membership(n: Nat, p: Pair[Nat, Nat]) {
    right_divisor_pair_list(n).contains(p) implies left_divisor_pair_list(n).contains(p)
} by {
    if right_divisor_pair_list(n).contains(p) {
        right_divisor_pair_list_contains_implies(n, p)
        divisor_list(n).contains(p.first)
        divisor_list(divisor_quotient(n, p.first)).contains(p.second)
        divisor_list_contains_implies(n, p.first)
        Nat.0 < p.first and p.first.divides(n)
        Nat.0 < p.first
        p.first.divides(n)
        divisor_list_contains_implies(divisor_quotient(n, p.first), p.second)
        Nat.0 < p.second and p.second.divides(divisor_quotient(n, p.first))
        Nat.0 < p.second
        p.second.divides(divisor_quotient(n, p.first))
        divisor_pair_product_in_divisor_list(n, p.first, p.second)
        divisor_list(n).contains(p.first * p.second)
        p.first * p.second != Nat.0
        Nat.0 < p.first * p.second
        p.first.divides(p.first * p.second)
        divisor_list_contains_of(p.first * p.second, p.first)
        divisor_list(p.first * p.second).contains(p.first)
        left_divisor_pair_list_contains(n, p.first * p.second, p.first)
        left_divisor_pair_list(n).contains(
            Pair.new(p.first, divisor_quotient(p.first * p.second, p.first)))
        divisor_quotient_cofactor(p.first * p.second, p.first)
        p.first * divisor_quotient(p.first * p.second, p.first) = p.first * p.second
        Nat.0 < p.first
        p.first != Nat.0
        mul_cancel_left(p.first, divisor_quotient(p.first * p.second, p.first), p.second)
        divisor_quotient(p.first * p.second, p.first) = p.second
        Pair.new(p.first, divisor_quotient(p.first * p.second, p.first)) =
            Pair.new(p.first, p.second)
        Pair.new(p.first, p.second) = p
        left_divisor_pair_list(n).contains(p)
    }
}

/// The canonical right and left divisor-pair lists contain the same pairs.
theorem right_left_pair_lists_same_contains(n: Nat, p: Pair[Nat, Nat]) {
    right_divisor_pair_list(n).contains(p) = left_divisor_pair_list(n).contains(p)
} by {
    if right_divisor_pair_list(n).contains(p) {
        right_pair_membership_implies_left_membership(n, p)
        left_divisor_pair_list(n).contains(p)
    }
    if left_divisor_pair_list(n).contains(p) {
        left_pair_membership_implies_right_membership(n, p)
        right_divisor_pair_list(n).contains(p)
    }
}

/// A left-block constructor over an explicit inner divisor list. This mirrors
/// `left_divisor_pair_block`, but lets bounded inner-sum proofs work over
/// `divisors_up_to`.
define left_divisor_pair_block_from(n: Nat, r: Nat, inner: List[Nat]) -> List[Pair[Nat, Nat]] {
    map(inner, left_divisor_pair_of(r))
}

/// The accepted left block is the explicit-inner block at the divisor list of
/// the represented product.
theorem left_divisor_pair_block_eq_from(n: Nat, r: Nat) {
    left_divisor_pair_block(n, r) = left_divisor_pair_block_from(n, r, divisor_list(r))
}

/// Cons equation for the explicit left-pair block. Keeping this as a small
/// lemma avoids repeatedly unfolding the mapped pair constructor in larger
/// bounded-sum inductions.
theorem left_divisor_pair_block_from_cons(n: Nat, r: Nat, head: Nat, tail: List[Nat]) {
    left_divisor_pair_block_from(n, r, List.cons(head, tail)) =
        List.cons(Pair.new(head, divisor_quotient(r, head)),
            left_divisor_pair_block_from(n, r, tail))
} by {
    let make_pair = left_divisor_pair_of(r)
    make_pair(head) = Pair.new(head, divisor_quotient(r, head))
    left_divisor_pair_block_from(n, r, List.cons(head, tail)) =
        map(List.cons(head, tail), make_pair)
    map(List.cons(head, tail), make_pair) =
        List.cons(make_pair(head), map(tail, make_pair))
    map(List.cons(head, tail), make_pair) =
        List.cons(Pair.new(head, divisor_quotient(r, head)), map(tail, make_pair))
    left_divisor_pair_block_from(n, r, tail) = map(tail, make_pair)
    left_divisor_pair_block_from(n, r, List.cons(head, tail)) =
        List.cons(Pair.new(head, divisor_quotient(r, head)),
            left_divisor_pair_block_from(n, r, tail))
}

/// A left-nested inner term is already the canonical pair term when `r | n` and
/// `d | r`, with the pair encoded as `(d, r/d)`.
theorem left_nested_inner_head_term_eq_pair_assoc_term(f: Nat -> Nat, g: Nat -> Nat,
        h: Nat -> Nat, n: Nat, r: Nat, d: Nat) {
    Nat.0 < n and r.divides(n) and d.divides(r) implies
        dirichlet_term(f, g, r)(d) * h(divisor_quotient(n, r)) =
            divisor_pair_assoc_term(f, g, h, n)(Pair.new(d, divisor_quotient(r, d)))
} by {
    if Nat.0 < n and r.divides(n) and d.divides(r) {
        let e: Nat = divisor_quotient(r, d)
        dirichlet_term_apply(f, g, r, d)
        dirichlet_term(f, g, r)(d) = f(d) * g(divisor_quotient(r, d))
        dirichlet_term(f, g, r)(d) = f(d) * g(e)
        divisor_quotient_cofactor(r, d)
        d * e = r
        divisor_quotient(n, d * e) = divisor_quotient(n, r)
        Pair.new(d, e).first = d
        Pair.new(d, e).second = e
        divisor_pair_assoc_term(f, g, h, n)(Pair.new(d, e)) =
            f(d) * g(e) * h(divisor_quotient(n, d * e))
        divisor_pair_assoc_term(f, g, h, n)(Pair.new(d, e)) =
            f(d) * g(e) * h(divisor_quotient(n, r))
        dirichlet_term(f, g, r)(d) * h(divisor_quotient(n, r)) =
            f(d) * g(e) * h(divisor_quotient(n, r))
        dirichlet_term(f, g, r)(d) * h(divisor_quotient(n, r)) =
            divisor_pair_assoc_term(f, g, h, n)(Pair.new(d, e))
        e = divisor_quotient(r, d)
        dirichlet_term(f, g, r)(d) * h(divisor_quotient(n, r)) =
            divisor_pair_assoc_term(f, g, h, n)(Pair.new(d, divisor_quotient(r, d)))
    }
}

/// Bounded left-nested inner-sum expansion over `divisors_up_to`: for fixed
/// `r | n`, the inner convolution sum scaled by the outer cofactor term is the
/// sum over the corresponding explicit left-pair block.
theorem left_nested_inner_sum_divisors_up_to_eq_pair_block_from_sum(f: Nat -> Nat,
        g: Nat -> Nat, h: Nat -> Nat, n: Nat, r: Nat, k: Nat) {
    Nat.0 < n and r.divides(n) implies
        sum(map(divisors_up_to(r, k), dirichlet_term(f, g, r))) * h(divisor_quotient(n, r)) =
            sum(map(left_divisor_pair_block_from(n, r, divisors_up_to(r, k)),
                divisor_pair_assoc_term(f, g, h, n)))
} by {
    let q: Nat = divisor_quotient(n, r)
    define pred(bound: Nat) -> Bool {
        Nat.0 < n and r.divides(n) implies
            sum(map(divisors_up_to(r, bound), dirichlet_term(f, g, r))) * h(q) =
                sum(map(left_divisor_pair_block_from(n, r, divisors_up_to(r, bound)),
                    divisor_pair_assoc_term(f, g, h, n)))
    }
    divisors_up_to_zero(r)
    divisors_up_to(r, Nat.0) = List.nil[Nat]
    map[Nat, Nat](List.nil[Nat], dirichlet_term(f, g, r)) = List.nil[Nat]
    sum[Nat](List.nil[Nat]) = Nat.0
    Nat.0 * h(q) = Nat.0
    left_divisor_pair_block_from(n, r, List.nil[Nat]) = List.nil[Pair[Nat, Nat]]
    map[Pair[Nat, Nat], Nat](List.nil[Pair[Nat, Nat]], divisor_pair_assoc_term(f, g, h, n)) =
        List.nil[Nat]
    pred(Nat.0)
    forall(j: Nat) {
        if pred(j) {
            pred(j) = (Nat.0 < n and r.divides(n) implies
                sum(map(divisors_up_to(r, j), dirichlet_term(f, g, r))) * h(q) =
                    sum(map(left_divisor_pair_block_from(n, r, divisors_up_to(r, j)),
                        divisor_pair_assoc_term(f, g, h, n))))
            if Nat.0 < n and r.divides(n) {
                if j.suc.divides(r) {
                    divisors_up_to_suc_yes(r, j)
                    divisors_up_to(r, j.suc) = List.cons(j.suc, divisors_up_to(r, j))
                    map(List.cons(j.suc, divisors_up_to(r, j)), dirichlet_term(f, g, r)) =
                        List.cons(dirichlet_term(f, g, r)(j.suc),
                            map(divisors_up_to(r, j), dirichlet_term(f, g, r)))
                    sum(map(divisors_up_to(r, j.suc), dirichlet_term(f, g, r))) =
                        sum(List.cons(dirichlet_term(f, g, r)(j.suc),
                            map(divisors_up_to(r, j), dirichlet_term(f, g, r))))
                    sum_cons_nat(dirichlet_term(f, g, r)(j.suc),
                        map(divisors_up_to(r, j), dirichlet_term(f, g, r)))
                    sum(List.cons(dirichlet_term(f, g, r)(j.suc),
                            map(divisors_up_to(r, j), dirichlet_term(f, g, r)))) =
                        dirichlet_term(f, g, r)(j.suc) +
                            sum(map(divisors_up_to(r, j), dirichlet_term(f, g, r)))
                    sum(map(divisors_up_to(r, j.suc), dirichlet_term(f, g, r))) =
                        dirichlet_term(f, g, r)(j.suc) +
                            sum(map(divisors_up_to(r, j), dirichlet_term(f, g, r)))
                    sum(map(divisors_up_to(r, j.suc), dirichlet_term(f, g, r))) * h(q) =
                        (dirichlet_term(f, g, r)(j.suc) +
                            sum(map(divisors_up_to(r, j), dirichlet_term(f, g, r)))) * h(q)
                    (dirichlet_term(f, g, r)(j.suc) +
                            sum(map(divisors_up_to(r, j), dirichlet_term(f, g, r)))) * h(q) =
                        dirichlet_term(f, g, r)(j.suc) * h(q) +
                            sum(map(divisors_up_to(r, j), dirichlet_term(f, g, r))) * h(q)
                    pred(j)
                    sum(map(divisors_up_to(r, j), dirichlet_term(f, g, r))) * h(q) =
                        sum(map(left_divisor_pair_block_from(n, r, divisors_up_to(r, j)),
                            divisor_pair_assoc_term(f, g, h, n)))
                    left_divisor_pair_block_from_cons(n, r, j.suc, divisors_up_to(r, j))
                    left_divisor_pair_block_from(n, r, List.cons(j.suc, divisors_up_to(r, j))) =
                        List.cons(Pair.new(j.suc, divisor_quotient(r, j.suc)),
                            left_divisor_pair_block_from(n, r, divisors_up_to(r, j)))
                    left_divisor_pair_block_from(n, r, divisors_up_to(r, j.suc)) =
                        List.cons(Pair.new(j.suc, divisor_quotient(r, j.suc)),
                            left_divisor_pair_block_from(n, r, divisors_up_to(r, j)))
                    map(left_divisor_pair_block_from(n, r, divisors_up_to(r, j.suc)),
                            divisor_pair_assoc_term(f, g, h, n)) =
                        List.cons(divisor_pair_assoc_term(f, g, h, n)(
                                Pair.new(j.suc, divisor_quotient(r, j.suc))),
                            map(left_divisor_pair_block_from(n, r, divisors_up_to(r, j)),
                                divisor_pair_assoc_term(f, g, h, n)))
                    sum(map(left_divisor_pair_block_from(n, r, divisors_up_to(r, j.suc)),
                            divisor_pair_assoc_term(f, g, h, n))) =
                        sum(List.cons(divisor_pair_assoc_term(f, g, h, n)(
                                Pair.new(j.suc, divisor_quotient(r, j.suc))),
                            map(left_divisor_pair_block_from(n, r, divisors_up_to(r, j)),
                                divisor_pair_assoc_term(f, g, h, n))))
                    sum_cons_nat(divisor_pair_assoc_term(f, g, h, n)(
                            Pair.new(j.suc, divisor_quotient(r, j.suc))),
                        map(left_divisor_pair_block_from(n, r, divisors_up_to(r, j)),
                            divisor_pair_assoc_term(f, g, h, n)))
                    sum(List.cons(divisor_pair_assoc_term(f, g, h, n)(
                                Pair.new(j.suc, divisor_quotient(r, j.suc))),
                            map(left_divisor_pair_block_from(n, r, divisors_up_to(r, j)),
                                divisor_pair_assoc_term(f, g, h, n)))) =
                        divisor_pair_assoc_term(f, g, h, n)(
                            Pair.new(j.suc, divisor_quotient(r, j.suc))) +
                            sum(map(left_divisor_pair_block_from(n, r, divisors_up_to(r, j)),
                                divisor_pair_assoc_term(f, g, h, n)))
                    sum(map(left_divisor_pair_block_from(n, r, divisors_up_to(r, j.suc)),
                            divisor_pair_assoc_term(f, g, h, n))) =
                        divisor_pair_assoc_term(f, g, h, n)(
                            Pair.new(j.suc, divisor_quotient(r, j.suc))) +
                            sum(map(left_divisor_pair_block_from(n, r, divisors_up_to(r, j)),
                                divisor_pair_assoc_term(f, g, h, n)))
                    left_nested_inner_head_term_eq_pair_assoc_term(f, g, h, n, r, j.suc)
                    dirichlet_term(f, g, r)(j.suc) * h(divisor_quotient(n, r)) =
                        divisor_pair_assoc_term(f, g, h, n)(
                            Pair.new(j.suc, divisor_quotient(r, j.suc)))
                    q = divisor_quotient(n, r)
                    dirichlet_term(f, g, r)(j.suc) * h(q) =
                        divisor_pair_assoc_term(f, g, h, n)(
                            Pair.new(j.suc, divisor_quotient(r, j.suc)))
                    sum(map(divisors_up_to(r, j.suc), dirichlet_term(f, g, r))) * h(q) =
                        sum(map(left_divisor_pair_block_from(n, r, divisors_up_to(r, j.suc)),
                            divisor_pair_assoc_term(f, g, h, n)))
                } else {
                    not j.suc.divides(r)
                    divisors_up_to_suc_no(r, j)
                    divisors_up_to(r, j.suc) = divisors_up_to(r, j)
                    pred(j)
                    sum(map(divisors_up_to(r, j), dirichlet_term(f, g, r))) * h(q) =
                        sum(map(left_divisor_pair_block_from(n, r, divisors_up_to(r, j)),
                            divisor_pair_assoc_term(f, g, h, n)))
                    sum(map(divisors_up_to(r, j.suc), dirichlet_term(f, g, r))) * h(q) =
                        sum(map(left_divisor_pair_block_from(n, r, divisors_up_to(r, j.suc)),
                            divisor_pair_assoc_term(f, g, h, n)))
                }
            }
            pred(j.suc) = (Nat.0 < n and r.divides(n) implies
                sum(map(divisors_up_to(r, j.suc), dirichlet_term(f, g, r))) * h(q) =
                    sum(map(left_divisor_pair_block_from(n, r, divisors_up_to(r, j.suc)),
                        divisor_pair_assoc_term(f, g, h, n))))
            pred(j.suc)
        }
    }
    forall(bound: Nat) { pred(bound) implies pred(bound.suc) }
    pred(Nat.0) and forall(bound: Nat) { pred(bound) implies pred(bound.suc) }
    alt_induction(pred)
    forall(bound: Nat) { pred(bound) }
    pred(k)
}

/// Sum form over the represented product's divisor list for the left-nested
/// expansion.
theorem left_nested_inner_sum_eq_pair_block_from_sum(f: Nat -> Nat, g: Nat -> Nat,
        h: Nat -> Nat, n: Nat, r: Nat) {
    Nat.0 < n and r.divides(n) implies
        sum(map(divisor_list(r), dirichlet_term(f, g, r))) * h(divisor_quotient(n, r)) =
            sum(map(left_divisor_pair_block_from(n, r, divisor_list(r)),
                divisor_pair_assoc_term(f, g, h, n)))
} by {
    if Nat.0 < n and r.divides(n) {
        left_nested_inner_sum_divisors_up_to_eq_pair_block_from_sum(f, g, h, n, r, r)
        divisor_list(r) = divisors_up_to(r, r)
        sum(map(divisor_list(r), dirichlet_term(f, g, r))) * h(divisor_quotient(n, r)) =
            sum(map(left_divisor_pair_block_from(n, r, divisor_list(r)),
                divisor_pair_assoc_term(f, g, h, n)))
    }
}

/// For a genuine outer divisor `r | n`, the `r`-summand of
/// `(f * g) * h` expands to the canonical sum over the left pair block for `r`.
theorem left_nested_outer_term_eq_pair_block_sum(f: Nat -> Nat, g: Nat -> Nat,
        h: Nat -> Nat, n: Nat, r: Nat) {
    Nat.0 < n and divisor_list(n).contains(r) implies
        dirichlet_term(dirichlet_convolve(f, g), h, n)(r) =
            sum(map(left_divisor_pair_block(n, r), divisor_pair_assoc_term(f, g, h, n)))
} by {
    if Nat.0 < n and divisor_list(n).contains(r) {
        divisor_list_contains_implies(n, r)
        Nat.0 < r and r.divides(n)
        r.divides(n)
        dirichlet_term_apply(dirichlet_convolve(f, g), h, n, r)
        dirichlet_term(dirichlet_convolve(f, g), h, n)(r) =
            dirichlet_convolve(f, g)(r) * h(divisor_quotient(n, r))
        dirichlet_convolve_apply(f, g, r)
        dirichlet_convolve(f, g)(r) = sum(map(divisor_list(r), dirichlet_term(f, g, r)))
        dirichlet_term(dirichlet_convolve(f, g), h, n)(r) =
            sum(map(divisor_list(r), dirichlet_term(f, g, r))) * h(divisor_quotient(n, r))
        left_nested_inner_sum_eq_pair_block_from_sum(f, g, h, n, r)
        sum(map(divisor_list(r), dirichlet_term(f, g, r))) * h(divisor_quotient(n, r)) =
            sum(map(left_divisor_pair_block_from(n, r, divisor_list(r)),
                divisor_pair_assoc_term(f, g, h, n)))
        left_divisor_pair_block_eq_from(n, r)
        left_divisor_pair_block(n, r) = left_divisor_pair_block_from(n, r, divisor_list(r))
        sum(map(left_divisor_pair_block(n, r), divisor_pair_assoc_term(f, g, h, n))) =
            sum(map(left_divisor_pair_block_from(n, r, divisor_list(r)),
                divisor_pair_assoc_term(f, g, h, n)))
        dirichlet_term(dirichlet_convolve(f, g), h, n)(r) =
            sum(map(left_divisor_pair_block(n, r), divisor_pair_assoc_term(f, g, h, n)))
    }
}

/// Bounded outer expansion over `divisors_up_to`: the left-nested convolution
/// summands indexed by outer divisors up to `k` flatten to the accepted left
/// divisor-pair list over the same bound.
theorem left_nested_outer_sum_divisors_up_to_eq_pair_list_from_sum(f: Nat -> Nat,
        g: Nat -> Nat, h: Nat -> Nat, n: Nat, k: Nat) {
    Nat.0 < n implies
        sum(map(divisors_up_to(n, k), dirichlet_term(dirichlet_convolve(f, g), h, n))) =
            sum(map(left_divisor_pair_list_from(n, divisors_up_to(n, k)),
                divisor_pair_assoc_term(f, g, h, n)))
} by {
    define pred(bound: Nat) -> Bool {
        Nat.0 < n implies
            sum(map(divisors_up_to(n, bound), dirichlet_term(dirichlet_convolve(f, g), h, n))) =
                sum(map(left_divisor_pair_list_from(n, divisors_up_to(n, bound)),
                    divisor_pair_assoc_term(f, g, h, n)))
    }
    divisors_up_to_zero(n)
    divisors_up_to(n, Nat.0) = List.nil[Nat]
    map[Nat, Nat](List.nil[Nat], dirichlet_term(dirichlet_convolve(f, g), h, n)) = List.nil[Nat]
    sum[Nat](List.nil[Nat]) = Nat.0
    left_divisor_pair_list_from(n, List.nil[Nat]) = List.nil[Pair[Nat, Nat]]
    map[Pair[Nat, Nat], Nat](List.nil[Pair[Nat, Nat]], divisor_pair_assoc_term(f, g, h, n)) =
        List.nil[Nat]
    pred(Nat.0)
    forall(j: Nat) {
        if pred(j) {
            pred(j) = (Nat.0 < n implies
                sum(map(divisors_up_to(n, j), dirichlet_term(dirichlet_convolve(f, g), h, n))) =
                    sum(map(left_divisor_pair_list_from(n, divisors_up_to(n, j)),
                        divisor_pair_assoc_term(f, g, h, n))))
            if Nat.0 < n {
                if j.suc.divides(n) {
                    divisors_up_to_suc_yes(n, j)
                    divisors_up_to(n, j.suc) = List.cons(j.suc, divisors_up_to(n, j))
                    map(List.cons(j.suc, divisors_up_to(n, j)),
                            dirichlet_term(dirichlet_convolve(f, g), h, n)) =
                        List.cons(dirichlet_term(dirichlet_convolve(f, g), h, n)(j.suc),
                            map(divisors_up_to(n, j), dirichlet_term(dirichlet_convolve(f, g), h, n)))
                    sum(map(divisors_up_to(n, j.suc),
                            dirichlet_term(dirichlet_convolve(f, g), h, n))) =
                        sum(List.cons(dirichlet_term(dirichlet_convolve(f, g), h, n)(j.suc),
                            map(divisors_up_to(n, j), dirichlet_term(dirichlet_convolve(f, g), h, n))))
                    sum_cons_nat(dirichlet_term(dirichlet_convolve(f, g), h, n)(j.suc),
                        map(divisors_up_to(n, j), dirichlet_term(dirichlet_convolve(f, g), h, n)))
                    sum(map(divisors_up_to(n, j.suc),
                            dirichlet_term(dirichlet_convolve(f, g), h, n))) =
                        dirichlet_term(dirichlet_convolve(f, g), h, n)(j.suc) +
                            sum(map(divisors_up_to(n, j),
                                dirichlet_term(dirichlet_convolve(f, g), h, n)))
                    divisor_list_contains_of(n, j.suc)
                    divisor_list(n).contains(j.suc)
                    left_nested_outer_term_eq_pair_block_sum(f, g, h, n, j.suc)
                    dirichlet_term(dirichlet_convolve(f, g), h, n)(j.suc) =
                        sum(map(left_divisor_pair_block(n, j.suc), divisor_pair_assoc_term(f, g, h, n)))
                    pred(j)
                    sum(map(divisors_up_to(n, j), dirichlet_term(dirichlet_convolve(f, g), h, n))) =
                        sum(map(left_divisor_pair_list_from(n, divisors_up_to(n, j)),
                            divisor_pair_assoc_term(f, g, h, n)))
                    left_divisor_pair_list_from(n, divisors_up_to(n, j.suc)) =
                        left_divisor_pair_block(n, j.suc) +
                            left_divisor_pair_list_from(n, divisors_up_to(n, j))
                    map_add(left_divisor_pair_block(n, j.suc),
                        left_divisor_pair_list_from(n, divisors_up_to(n, j)),
                        divisor_pair_assoc_term(f, g, h, n))
                    map(left_divisor_pair_block(n, j.suc) +
                            left_divisor_pair_list_from(n, divisors_up_to(n, j)),
                            divisor_pair_assoc_term(f, g, h, n)) =
                        map(left_divisor_pair_block(n, j.suc), divisor_pair_assoc_term(f, g, h, n)) +
                            map(left_divisor_pair_list_from(n, divisors_up_to(n, j)),
                                divisor_pair_assoc_term(f, g, h, n))
                    sum_add(map(left_divisor_pair_block(n, j.suc), divisor_pair_assoc_term(f, g, h, n)),
                        map(left_divisor_pair_list_from(n, divisors_up_to(n, j)),
                            divisor_pair_assoc_term(f, g, h, n)))
                    sum(map(left_divisor_pair_list_from(n, divisors_up_to(n, j.suc)),
                            divisor_pair_assoc_term(f, g, h, n))) =
                        sum(map(left_divisor_pair_block(n, j.suc), divisor_pair_assoc_term(f, g, h, n))) +
                            sum(map(left_divisor_pair_list_from(n, divisors_up_to(n, j)),
                                divisor_pair_assoc_term(f, g, h, n)))
                    sum(map(divisors_up_to(n, j.suc),
                            dirichlet_term(dirichlet_convolve(f, g), h, n))) =
                        sum(map(left_divisor_pair_list_from(n, divisors_up_to(n, j.suc)),
                            divisor_pair_assoc_term(f, g, h, n)))
                } else {
                    not j.suc.divides(n)
                    divisors_up_to_suc_no(n, j)
                    divisors_up_to(n, j.suc) = divisors_up_to(n, j)
                    pred(j)
                    sum(map(divisors_up_to(n, j), dirichlet_term(dirichlet_convolve(f, g), h, n))) =
                        sum(map(left_divisor_pair_list_from(n, divisors_up_to(n, j)),
                            divisor_pair_assoc_term(f, g, h, n)))
                    sum(map(divisors_up_to(n, j.suc), dirichlet_term(dirichlet_convolve(f, g), h, n))) =
                        sum(map(left_divisor_pair_list_from(n, divisors_up_to(n, j.suc)),
                            divisor_pair_assoc_term(f, g, h, n)))
                }
            }
            pred(j.suc) = (Nat.0 < n implies
                sum(map(divisors_up_to(n, j.suc), dirichlet_term(dirichlet_convolve(f, g), h, n))) =
                    sum(map(left_divisor_pair_list_from(n, divisors_up_to(n, j.suc)),
                        divisor_pair_assoc_term(f, g, h, n))))
            pred(j.suc)
        }
    }
    forall(bound: Nat) { pred(bound) implies pred(bound.suc) }
    pred(Nat.0) and forall(bound: Nat) { pred(bound) implies pred(bound.suc) }
    alt_induction(pred)
    forall(bound: Nat) { pred(bound) }
    pred(k)
}

/// One-sided fixed-positive-`n` expansion of the left-nested Dirichlet
/// convolution over the canonical left divisor-pair list.
theorem left_nested_convolve_eq_left_pair_list_sum(f: Nat -> Nat, g: Nat -> Nat,
        h: Nat -> Nat, n: Nat) {
    Nat.0 < n implies
        dirichlet_convolve(dirichlet_convolve(f, g), h)(n) =
            sum(map(left_divisor_pair_list(n), divisor_pair_assoc_term(f, g, h, n)))
} by {
    if Nat.0 < n {
        dirichlet_convolve_apply(dirichlet_convolve(f, g), h, n)
        dirichlet_convolve(dirichlet_convolve(f, g), h)(n) =
            sum(map(divisor_list(n), dirichlet_term(dirichlet_convolve(f, g), h, n)))
        left_nested_outer_sum_divisors_up_to_eq_pair_list_from_sum(f, g, h, n, n)
        divisor_list(n) = divisors_up_to(n, n)
        sum(map(divisor_list(n), dirichlet_term(dirichlet_convolve(f, g), h, n))) =
            sum(map(left_divisor_pair_list_from(n, divisor_list(n)),
                divisor_pair_assoc_term(f, g, h, n)))
        left_divisor_pair_list(n) = left_divisor_pair_list_from(n, divisor_list(n))
        sum(map(left_divisor_pair_list(n), divisor_pair_assoc_term(f, g, h, n))) =
            sum(map(left_divisor_pair_list_from(n, divisor_list(n)),
                divisor_pair_assoc_term(f, g, h, n)))
        dirichlet_convolve(dirichlet_convolve(f, g), h)(n) =
            sum(map(left_divisor_pair_list(n), divisor_pair_assoc_term(f, g, h, n)))
    }
}

/// Each fixed-left right divisor-pair block is unique.
theorem right_divisor_pair_block_unique(n: Nat, d: Nat) {
    right_divisor_pair_block(n, d).is_unique
} by {
    let make_pair = function(e: Nat) { Pair.new(d, e) }
    forall(e1: Nat, e2: Nat) {
        if make_pair(e1) = make_pair(e2) {
            make_pair(e1) = Pair.new(d, e1)
            make_pair(e2) = Pair.new(d, e2)
            Pair.new(d, e1) = Pair.new(d, e2)
            Pair.new(d, e1).second = e1
            Pair.new(d, e2).second = e2
            e1 = e2
        }
    }
    is_injective_fn(make_pair)
    divisor_list_is_unique(divisor_quotient(n, d))
    injective_map_is_unique[Nat, Pair[Nat, Nat]](divisor_list(divisor_quotient(n, d)), make_pair)
    map[Nat, Pair[Nat, Nat]](divisor_list(divisor_quotient(n, d)), make_pair).is_unique
    right_divisor_pair_block(n, d) = map(divisor_list(divisor_quotient(n, d)), make_pair)
    right_divisor_pair_block(n, d).is_unique
}

/// The bounded canonical right divisor-pair list is unique at every bound.
theorem right_divisor_pair_list_from_divisors_up_to_unique(n: Nat, k: Nat) {
    right_divisor_pair_list_from(n, divisors_up_to(n, k)).is_unique
} by {
    define pred(bound: Nat) -> Bool {
        right_divisor_pair_list_from(n, divisors_up_to(n, bound)).is_unique
    }
    divisors_up_to_zero(n)
    divisors_up_to(n, Nat.0) = List.nil[Nat]
    right_divisor_pair_list_from(n, divisors_up_to(n, Nat.0)) = List.nil[Pair[Nat, Nat]]
    List.nil[Pair[Nat, Nat]].is_unique
    pred(Nat.0)
    forall(j: Nat) {
        if pred(j) {
            if j.suc.divides(n) {
                divisors_up_to_suc_yes(n, j)
                divisors_up_to(n, j.suc) = List.cons(j.suc, divisors_up_to(n, j))
                right_divisor_pair_list_from(n, divisors_up_to(n, j.suc)) =
                    right_divisor_pair_block(n, j.suc) +
                        right_divisor_pair_list_from(n, divisors_up_to(n, j))
                right_divisor_pair_block_unique(n, j.suc)
                right_divisor_pair_block(n, j.suc).is_unique
                pred(j)
                right_divisor_pair_list_from(n, divisors_up_to(n, j)).is_unique
                forall(p: Pair[Nat, Nat]) {
                    if right_divisor_pair_block(n, j.suc).contains(p) and
                            right_divisor_pair_list_from(n, divisors_up_to(n, j)).contains(p) {
                        right_divisor_pair_block_contains_implies(n, j.suc, p)
                        p.first = j.suc
                        right_divisor_pair_list_from_divisors_up_to_contains_implies(n, j, p)
                        divisors_up_to(n, j).contains(p.first)
                        divisors_up_to(n, j).contains(j.suc)
                        divisors_up_to_member(n, j, j.suc)
                        Nat.0 < j.suc and j.suc <= j and j.suc.divides(n)
                        j.suc <= j
                        false
                    }
                }
                unique_list_sum(right_divisor_pair_block(n, j.suc),
                    right_divisor_pair_list_from(n, divisors_up_to(n, j)))
                (right_divisor_pair_block(n, j.suc) +
                    right_divisor_pair_list_from(n, divisors_up_to(n, j))).is_unique
                right_divisor_pair_list_from(n, divisors_up_to(n, j.suc)).is_unique
            } else {
                not j.suc.divides(n)
                divisors_up_to_suc_no(n, j)
                divisors_up_to(n, j.suc) = divisors_up_to(n, j)
                pred(j)
                right_divisor_pair_list_from(n, divisors_up_to(n, j.suc)).is_unique
            }
            pred(j.suc)
        }
    }
    forall(bound: Nat) { pred(bound) implies pred(bound.suc) }
    pred(Nat.0) and forall(bound: Nat) { pred(bound) implies pred(bound.suc) }
    alt_induction(pred)
    forall(bound: Nat) { pred(bound) }
    pred(k)
}

/// The canonical right divisor-pair list is unique.
theorem right_divisor_pair_list_unique(n: Nat) {
    right_divisor_pair_list(n).is_unique
} by {
    right_divisor_pair_list_from_divisors_up_to_unique(n, n)
    right_divisor_pair_list_from(n, divisors_up_to(n, n)).is_unique
    divisor_list(n) = divisors_up_to(n, n)
    right_divisor_pair_list(n) = right_divisor_pair_list_from(n, divisor_list(n))
    right_divisor_pair_list(n).is_unique
}

/// Each left divisor-pair block is unique.
theorem left_divisor_pair_block_unique(n: Nat, r: Nat) {
    left_divisor_pair_block(n, r).is_unique
} by {
    let make_pair = left_divisor_pair_of(r)
    forall(d1: Nat, d2: Nat) {
        if make_pair(d1) = make_pair(d2) {
            make_pair(d1) = Pair.new(d1, divisor_quotient(r, d1))
            make_pair(d2) = Pair.new(d2, divisor_quotient(r, d2))
            Pair.new(d1, divisor_quotient(r, d1)) = Pair.new(d2, divisor_quotient(r, d2))
            Pair.new(d1, divisor_quotient(r, d1)).first = d1
            Pair.new(d2, divisor_quotient(r, d2)).first = d2
            d1 = d2
        }
    }
    is_injective_fn(make_pair)
    divisor_list_is_unique(r)
    injective_map_is_unique[Nat, Pair[Nat, Nat]](divisor_list(r), make_pair)
    map[Nat, Pair[Nat, Nat]](divisor_list(r), make_pair).is_unique
    left_divisor_pair_block_eq_from(n, r)
    left_divisor_pair_block(n, r) = left_divisor_pair_block_from(n, r, divisor_list(r))
    left_divisor_pair_block_from(n, r, divisor_list(r)) = map(divisor_list(r), make_pair)
    left_divisor_pair_block(n, r).is_unique
}

/// The bounded canonical left divisor-pair list is unique at every bound.
theorem left_divisor_pair_list_from_divisors_up_to_unique(n: Nat, k: Nat) {
    left_divisor_pair_list_from(n, divisors_up_to(n, k)).is_unique
} by {
    define pred(bound: Nat) -> Bool {
        left_divisor_pair_list_from(n, divisors_up_to(n, bound)).is_unique
    }
    divisors_up_to_zero(n)
    divisors_up_to(n, Nat.0) = List.nil[Nat]
    left_divisor_pair_list_from(n, divisors_up_to(n, Nat.0)) = List.nil[Pair[Nat, Nat]]
    List.nil[Pair[Nat, Nat]].is_unique
    pred(Nat.0)
    forall(j: Nat) {
        if pred(j) {
            if j.suc.divides(n) {
                divisors_up_to_suc_yes(n, j)
                divisors_up_to(n, j.suc) = List.cons(j.suc, divisors_up_to(n, j))
                left_divisor_pair_list_from(n, divisors_up_to(n, j.suc)) =
                    left_divisor_pair_block(n, j.suc) +
                        left_divisor_pair_list_from(n, divisors_up_to(n, j))
                left_divisor_pair_block_unique(n, j.suc)
                left_divisor_pair_block(n, j.suc).is_unique
                pred(j)
                left_divisor_pair_list_from(n, divisors_up_to(n, j)).is_unique
                forall(p: Pair[Nat, Nat]) {
                    if left_divisor_pair_block(n, j.suc).contains(p) and
                            left_divisor_pair_list_from(n, divisors_up_to(n, j)).contains(p) {
                        left_divisor_pair_block_contains_implies_product(n, j.suc, p)
                        p.first * p.second = j.suc
                        left_divisor_pair_list_from_divisors_up_to_contains_implies(n, j, p)
                        divisors_up_to(n, j).contains(p.first * p.second)
                        divisors_up_to(n, j).contains(j.suc)
                        divisors_up_to_member(n, j, j.suc)
                        Nat.0 < j.suc and j.suc <= j and j.suc.divides(n)
                        j.suc <= j
                        false
                    }
                }
                unique_list_sum(left_divisor_pair_block(n, j.suc),
                    left_divisor_pair_list_from(n, divisors_up_to(n, j)))
                (left_divisor_pair_block(n, j.suc) +
                    left_divisor_pair_list_from(n, divisors_up_to(n, j))).is_unique
                left_divisor_pair_list_from(n, divisors_up_to(n, j.suc)).is_unique
            } else {
                not j.suc.divides(n)
                divisors_up_to_suc_no(n, j)
                divisors_up_to(n, j.suc) = divisors_up_to(n, j)
                pred(j)
                left_divisor_pair_list_from(n, divisors_up_to(n, j.suc)).is_unique
            }
            pred(j.suc)
        }
    }
    forall(bound: Nat) { pred(bound) implies pred(bound.suc) }
    pred(Nat.0) and forall(bound: Nat) { pred(bound) implies pred(bound.suc) }
    alt_induction(pred)
    forall(bound: Nat) { pred(bound) }
    pred(k)
}

/// The canonical left divisor-pair list is unique.
theorem left_divisor_pair_list_unique(n: Nat) {
    left_divisor_pair_list(n).is_unique
} by {
    left_divisor_pair_list_from_divisors_up_to_unique(n, n)
    left_divisor_pair_list_from(n, divisors_up_to(n, n)).is_unique
    divisor_list(n) = divisors_up_to(n, n)
    left_divisor_pair_list(n) = left_divisor_pair_list_from(n, divisor_list(n))
    left_divisor_pair_list(n).is_unique
}

/// The right and left canonical divisor-pair lists are unique.
theorem right_left_pair_lists_unique(n: Nat) {
    right_divisor_pair_list(n).is_unique and left_divisor_pair_list(n).is_unique
} by {
    right_divisor_pair_list_unique(n)
    left_divisor_pair_list_unique(n)
}

/// The two canonical pair lists yield the same sum of canonical associativity
/// terms because they are unique lists with the same pair membership.
theorem right_left_pair_list_assoc_term_sums_eq(f: Nat -> Nat, g: Nat -> Nat,
        h: Nat -> Nat, n: Nat) {
    sum(map(right_divisor_pair_list(n), divisor_pair_assoc_term(f, g, h, n))) =
        sum(map(left_divisor_pair_list(n), divisor_pair_assoc_term(f, g, h, n)))
} by {
    divisor_list_is_unique(n)
    right_left_pair_lists_unique(n)
    right_divisor_pair_list(n).is_unique
    left_divisor_pair_list(n).is_unique
    forall(p: Pair[Nat, Nat]) {
        right_left_pair_lists_same_contains(n, p)
        right_divisor_pair_list(n).contains(p) = left_divisor_pair_list(n).contains(p)
    }
    unique_same_contains_map_sum_eq(right_divisor_pair_list(n), left_divisor_pair_list(n),
        divisor_pair_assoc_term(f, g, h, n))
}

/// Dirichlet convolution is associative on positive arguments.
theorem dirichlet_convolve_assoc_positive(f: Nat -> Nat, g: Nat -> Nat,
        h: Nat -> Nat, n: Nat) {
    Nat.0 < n implies
        dirichlet_convolve(f, dirichlet_convolve(g, h))(n) =
            dirichlet_convolve(dirichlet_convolve(f, g), h)(n)
} by {
    if Nat.0 < n {
        right_nested_convolve_eq_right_pair_list_sum(f, g, h, n)
        dirichlet_convolve(f, dirichlet_convolve(g, h))(n) =
            sum(map(right_divisor_pair_list(n), divisor_pair_assoc_term(f, g, h, n)))
        left_nested_convolve_eq_left_pair_list_sum(f, g, h, n)
        dirichlet_convolve(dirichlet_convolve(f, g), h)(n) =
            sum(map(left_divisor_pair_list(n), divisor_pair_assoc_term(f, g, h, n)))
        right_left_pair_list_assoc_term_sums_eq(f, g, h, n)
        sum(map(right_divisor_pair_list(n), divisor_pair_assoc_term(f, g, h, n))) =
            sum(map(left_divisor_pair_list(n), divisor_pair_assoc_term(f, g, h, n)))
        dirichlet_convolve(f, dirichlet_convolve(g, h))(n) =
            dirichlet_convolve(dirichlet_convolve(f, g), h)(n)
    }
}

/// Dirichlet convolution is associative.
theorem dirichlet_convolve_assoc(f: Nat -> Nat, g: Nat -> Nat, h: Nat -> Nat) {
    dirichlet_convolve(f, dirichlet_convolve(g, h)) =
        dirichlet_convolve(dirichlet_convolve(f, g), h)
} by {
    forall(n: Nat) {
        if n = Nat.0 {
            dirichlet_convolve_at_zero(f, dirichlet_convolve(g, h))
            dirichlet_convolve_at_zero(dirichlet_convolve(f, g), h)
            dirichlet_convolve(f, dirichlet_convolve(g, h))(n) = Nat.0
            dirichlet_convolve(dirichlet_convolve(f, g), h)(n) = Nat.0
            dirichlet_convolve(f, dirichlet_convolve(g, h))(n) =
                dirichlet_convolve(dirichlet_convolve(f, g), h)(n)
        } else {
            Nat.0 < n
            dirichlet_convolve_assoc_positive(f, g, h, n)
            dirichlet_convolve(f, dirichlet_convolve(g, h))(n) =
                dirichlet_convolve(dirichlet_convolve(f, g), h)(n)
        }
    }
}
