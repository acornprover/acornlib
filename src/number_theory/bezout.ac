from nat import Nat
from int import Int, abs, abs_from_nat, spans, spans_gcd
numerals Nat

/// Bezout's identity on the natural numbers: the gcd of two naturals is an
/// integer-linear combination of those naturals.
theorem nat_bezout(a: Nat, b: Nat) {
    exists(x: Int, y: Int) {
        x * Int.from_nat(a) + y * Int.from_nat(b) = Int.from_nat(a.gcd(b))
    }
} by {
    let ai = Int.from_nat(a)
    let bi = Int.from_nat(b)
    abs_from_nat(a)
    abs_from_nat(b)
    spans_gcd(ai, bi)
}
