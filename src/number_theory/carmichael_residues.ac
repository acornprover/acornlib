from number_theory.congruence import Nat, mod_add_mul
from nat import mod_of_zero, lt_suc_right, lt_not_ref, not_lt_zero, lt_or_lte,
    lte_imp_not_lt
numerals Nat

/// `0.mod(2) = 0`.
theorem zero_mod_two_zero {
    Nat.0.mod(Nat.2) = Nat.0
} by {
    mod_of_zero(Nat.2)
}

/// `2.mod(2) = 0`.
theorem two_mod_two_zero {
    Nat.2.mod(Nat.2) = Nat.0
} by {
    mod_add_mul(Nat.1, Nat.2, Nat.0)
    (Nat.1 * Nat.2 + Nat.0).mod(Nat.2) = Nat.0.mod(Nat.2)
    Nat.1 * Nat.2 = Nat.2
    Nat.2 + Nat.0 = Nat.2
    Nat.1 * Nat.2 + Nat.0 = Nat.2
    Nat.2.mod(Nat.2) = Nat.0.mod(Nat.2)
    mod_of_zero(Nat.2)
    Nat.0.mod(Nat.2) = Nat.0
    Nat.2.mod(Nat.2) = Nat.0
}

/// `4.mod(2) = 0`.
theorem four_mod_two_zero {
    Nat.4.mod(Nat.2) = Nat.0
} by {
    mod_add_mul(Nat.2, Nat.2, Nat.0)
    (Nat.2 * Nat.2 + Nat.0).mod(Nat.2) = Nat.0.mod(Nat.2)
    Nat.2 * Nat.2 = Nat.4
    Nat.4 + Nat.0 = Nat.4
    Nat.2 * Nat.2 + Nat.0 = Nat.4
    Nat.4.mod(Nat.2) = Nat.0.mod(Nat.2)
    mod_of_zero(Nat.2)
    Nat.0.mod(Nat.2) = Nat.0
    Nat.4.mod(Nat.2) = Nat.0
}

/// `6.mod(2) = 0`.
theorem six_mod_two_zero {
    Nat.6.mod(Nat.2) = Nat.0
} by {
    mod_add_mul(Nat.3, Nat.2, Nat.0)
    (Nat.3 * Nat.2 + Nat.0).mod(Nat.2) = Nat.0.mod(Nat.2)
    Nat.3 * Nat.2 = Nat.6
    Nat.6 + Nat.0 = Nat.6
    Nat.3 * Nat.2 + Nat.0 = Nat.6
    Nat.6.mod(Nat.2) = Nat.0.mod(Nat.2)
    mod_of_zero(Nat.2)
    Nat.0.mod(Nat.2) = Nat.0
    Nat.6.mod(Nat.2) = Nat.0
}

/// A residue below `4` congruent to `1` modulo `2` is `1` or `3`.
theorem residue_below_four_mod_two_one(r: Nat) {
    r < Nat.4 and r.mod(Nat.2) = Nat.1 implies (r = Nat.1 or r = Nat.3)
} by {
    if r < Nat.4 and r.mod(Nat.2) = Nat.1 {
        lt_suc_right(r, Nat.3)
        if r = Nat.3 {
            r = Nat.1 or r = Nat.3
        } else {
            r < Nat.3
            lt_suc_right(r, Nat.2)
            if r = Nat.2 {
                two_mod_two_zero
                Nat.2.mod(Nat.2) = Nat.0
                r.mod(Nat.2) = Nat.0
                r.mod(Nat.2) = Nat.1
                Nat.0 = Nat.1
                false
            } else {
                r < Nat.2
                lt_suc_right(r, Nat.1)
                if r = Nat.1 {
                    r = Nat.1 or r = Nat.3
                } else {
                    r < Nat.1
                    lt_suc_right(r, Nat.0)
                    r = Nat.0 or r < Nat.0
                    if r < Nat.0 {
                        not_lt_zero(r)
                        false
                    }
                    r = Nat.0
                    zero_mod_two_zero
                    Nat.0.mod(Nat.2) = Nat.0
                    r.mod(Nat.2) = Nat.0
                    r.mod(Nat.2) = Nat.1
                    Nat.0 = Nat.1
                    false
                }
            }
        }
    }
}

/// A residue in `[4, 8)` congruent to `1` modulo `2` is `5` or `7`.
theorem residue_from_four_mod_two_one(r: Nat) {
    Nat.4 <= r and r < Nat.8 and r.mod(Nat.2) = Nat.1
        implies (r = Nat.5 or r = Nat.7)
} by {
    if Nat.4 <= r and r < Nat.8 and r.mod(Nat.2) = Nat.1 {
        lt_suc_right(r, Nat.7)
        if r = Nat.7 {
            r = Nat.5 or r = Nat.7
        } else {
            r < Nat.7
            lt_suc_right(r, Nat.6)
            if r = Nat.6 {
                six_mod_two_zero
                Nat.6.mod(Nat.2) = Nat.0
                r.mod(Nat.2) = Nat.0
                r.mod(Nat.2) = Nat.1
                Nat.0 = Nat.1
                false
            } else {
                r < Nat.6
                lt_suc_right(r, Nat.5)
                if r = Nat.5 {
                    r = Nat.5 or r = Nat.7
                } else {
                    r < Nat.5
                    lt_suc_right(r, Nat.4)
                    if r = Nat.4 {
                        four_mod_two_zero
                        Nat.4.mod(Nat.2) = Nat.0
                        r.mod(Nat.2) = Nat.0
                        r.mod(Nat.2) = Nat.1
                        Nat.0 = Nat.1
                        false
                    } else {
                        r < Nat.4
                        lte_imp_not_lt(Nat.4, r)
                        false
                    }
                }
            }
        }
    }
}

/// A residue below `8` congruent to `1` modulo `2` is odd: `1`, `3`, `5` or `7`.
theorem residue_eight_mod_two_one_odd(r: Nat) {
    r < Nat.8 and r.mod(Nat.2) = Nat.1
        implies (r = Nat.1 or r = Nat.3 or r = Nat.5 or r = Nat.7)
} by {
    if r < Nat.8 and r.mod(Nat.2) = Nat.1 {
        lt_or_lte(r, Nat.4)
        if r < Nat.4 {
            residue_below_four_mod_two_one(r)
            r = Nat.1 or r = Nat.3
            r = Nat.1 or r = Nat.3 or r = Nat.5 or r = Nat.7
        } else {
            Nat.4 <= r
            residue_from_four_mod_two_one(r)
            r = Nat.5 or r = Nat.7
            r = Nat.1 or r = Nat.3 or r = Nat.5 or r = Nat.7
        }
    }
}
