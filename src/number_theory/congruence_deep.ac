/// Deepening of the congruence theory on the natural numbers: the
/// divisibility-of-difference characterization, the equivalence-relation
/// structure, the algebraic preservation laws, and the remainder
/// characterization. Most theorems restate the core results of
/// `number_theory.congruence` in their cleanest form. The genuinely new
/// content is the divisibility characterization, which needs an ordering
/// hypothesis on `Nat` because natural subtraction truncates.

from nat import add_sub
from number_theory.congruence import Nat, congr_mod_refl, congr_mod_symm, congr_mod_trans,
    congr_mod_add, congr_mod_mul, congr_mod_pow, mod_add_mul
from number_theory.congr_int import nat_congr_mod_iff_int_mod_rel, nat_congr_mod_imp_int_mod_rel,
    int_from_nat_sub_eq
from int import Int, abs, abs_from_nat, div_imp_div_abs
from zmod import int_mod_rel
from data.basic.relation_basic import is_reflexive, is_symmetric, is_transitive, is_equivalence,
    reflexive_symmetric_transitive_imp_equivalence

// ---------------------------------------------------------------------------
// (a) Characterization by divisibility of the difference.
//
// On the natural numbers, "a ≡ b (mod n) iff n | (a - b)" needs the ordering
// b <= a, because Nat subtraction truncates: without it, 3 | (1 - 5) would
// hold (1 - 5 truncates to 0) while 1 is not congruent to 5 modulo 3. The
// integer form below is the unconditional version of the characterization.
// ---------------------------------------------------------------------------

/// Congruence with the larger number on the left implies divisibility of the
/// difference by the modulus.
theorem congr_mod_imp_divides_diff(a: Nat, b: Nat, n: Nat) {
    b <= a and a.congr_mod(b, n) implies n.divides(a - b)
} by {
    if b <= a and a.congr_mod(b, n) {
        nat_congr_mod_imp_int_mod_rel(a, b, n)
        int_mod_rel(n, Int.from_nat(a), Int.from_nat(b))
        Int.from_nat(n).divides(Int.from_nat(a) - Int.from_nat(b))
        int_from_nat_sub_eq(a, b)
        Int.from_nat(a) - Int.from_nat(b) = Int.from_nat(a - b)
        Int.from_nat(n).divides(Int.from_nat(a - b))
        div_imp_div_abs(Int.from_nat(n), Int.from_nat(a - b))
        abs(Int.from_nat(n)).divides(abs(Int.from_nat(a - b)))
        abs_from_nat(n)
        abs(Int.from_nat(n)) = n
        abs_from_nat(a - b)
        abs(Int.from_nat(a - b)) = a - b
        n.divides(a - b)
    }
}

/// Divisibility of the difference by the modulus gives congruence, when the
/// larger number is on the left.
theorem divides_diff_imp_congr_mod(a: Nat, b: Nat, n: Nat) {
    b <= a and n.divides(a - b) implies a.congr_mod(b, n)
} by {
    if b <= a and n.divides(a - b) {
        n.divides(a - b) = exists(q: Nat) { n * q = a - b }
        let k: Nat satisfy { n * k = a - b }
        add_sub(a, b)
        a - b + b = a
        n * k + b = a
        mod_add_mul(k, n, b)
        (k * n + b).mod(n) = b.mod(n)
        k * n = n * k
        (n * k + b).mod(n) = b.mod(n)
        a.mod(n) = b.mod(n)
        a.congr_mod(b, n)
    }
}

/// The characterization of congruence by divisibility of the difference,
/// for a difference with the larger number on the left.
theorem congr_mod_iff_divides_diff(a: Nat, b: Nat, n: Nat) {
    b <= a implies (a.congr_mod(b, n) = n.divides(a - b))
} by {
    if b <= a {
        if a.congr_mod(b, n) {
            congr_mod_imp_divides_diff(a, b, n)
            n.divides(a - b)
        }
        if n.divides(a - b) {
            divides_diff_imp_congr_mod(a, b, n)
            a.congr_mod(b, n)
        }
        a.congr_mod(b, n) = n.divides(a - b)
    }
}

/// By symmetry, the characterization also holds with the roles of the two
/// numbers interchanged.
theorem congr_mod_iff_divides_diff_swapped(a: Nat, b: Nat, n: Nat) {
    a <= b implies (a.congr_mod(b, n) = n.divides(b - a))
} by {
    if a <= b {
        congr_mod_iff_divides_diff(b, a, n)
        b.congr_mod(a, n) = n.divides(b - a)
        if a.congr_mod(b, n) {
            congr_mod_symm(a, b, n)
            b.congr_mod(a, n)
            n.divides(b - a)
        }
        if n.divides(b - a) {
            b.congr_mod(a, n)
            congr_mod_symm(b, a, n)
            a.congr_mod(b, n)
        }
        a.congr_mod(b, n) = n.divides(b - a)
    }
}

/// The unconditional characterization on the integers: a is congruent to b
/// modulo n exactly when n divides the integer difference a - b.
theorem congr_mod_iff_int_divides_diff(a: Nat, b: Nat, n: Nat) {
    a.congr_mod(b, n) = Int.from_nat(n).divides(Int.from_nat(a) - Int.from_nat(b))
} by {
    nat_congr_mod_iff_int_mod_rel(a, b, n)
    int_mod_rel(n, Int.from_nat(a), Int.from_nat(b)) = Int.from_nat(n).divides(Int.from_nat(a) - Int.from_nat(b))
    a.congr_mod(b, n) = Int.from_nat(n).divides(Int.from_nat(a) - Int.from_nat(b))
}

// ---------------------------------------------------------------------------
// (b) Congruence is an equivalence relation.
// ---------------------------------------------------------------------------

/// Congruence modulo n, viewed as a binary relation on natural numbers.
define congr_mod_rel(n: Nat, a: Nat, b: Nat) -> Bool {
    a.congr_mod(b, n)
}

/// From congruence to the relation form.
theorem congr_mod_rel_intro(n: Nat, a: Nat, b: Nat) {
    a.congr_mod(b, n) implies congr_mod_rel(n, a, b)
} by {
    if a.congr_mod(b, n) {
        congr_mod_rel(n, a, b) = a.congr_mod(b, n)
        congr_mod_rel(n, a, b)
    }
}

/// From the relation form back to congruence.
theorem congr_mod_rel_apply(n: Nat, a: Nat, b: Nat) {
    congr_mod_rel(n, a, b) implies a.congr_mod(b, n)
} by {
    if congr_mod_rel(n, a, b) {
        congr_mod_rel(n, a, b) = a.congr_mod(b, n)
        a.congr_mod(b, n)
    }
}

/// Congruence modulo n is reflexive as a relation.
theorem congr_mod_rel_reflexive(n: Nat) {
    is_reflexive(congr_mod_rel(n))
} by {
    forall(a: Nat) {
        congr_mod_refl(a, n)
        congr_mod_rel_intro(n, a, a)
        congr_mod_rel(n, a, a)
    }
    is_reflexive(congr_mod_rel(n))
}

/// Congruence modulo n is symmetric as a relation.
theorem congr_mod_rel_symmetric(n: Nat) {
    is_symmetric(congr_mod_rel(n))
} by {
    forall(a: Nat, b: Nat) {
        if congr_mod_rel(n, a, b) {
            congr_mod_rel_apply(n, a, b)
            a.congr_mod(b, n)
            congr_mod_symm(a, b, n)
            b.congr_mod(a, n)
            congr_mod_rel_intro(n, b, a)
            congr_mod_rel(n, b, a)
        }
    }
    is_symmetric(congr_mod_rel(n))
}

/// Transitivity of the relation form of congruence, stated pointwise.
theorem congr_mod_rel_trans_imp(a: Nat, b: Nat, c: Nat, n: Nat) {
    congr_mod_rel(n, a, b) and congr_mod_rel(n, b, c) implies congr_mod_rel(n, a, c)
} by {
    if congr_mod_rel(n, a, b) and congr_mod_rel(n, b, c) {
        congr_mod_rel_apply(n, a, b)
        a.congr_mod(b, n)
        congr_mod_rel_apply(n, b, c)
        b.congr_mod(c, n)
        congr_mod_trans(a, b, c, n)
        a.congr_mod(c, n)
        congr_mod_rel_intro(n, a, c)
        congr_mod_rel(n, a, c)
    }
}

/// Congruence modulo n is transitive as a relation.
theorem congr_mod_rel_transitive(n: Nat) {
    is_transitive(congr_mod_rel(n))
} by {
    let r: (Nat, Nat) -> Bool = congr_mod_rel(n)
    forall(a: Nat, b: Nat, c: Nat) {
        if r(a, b) and r(b, c) {
            r(b, c) = congr_mod_rel(n, b, c)
            congr_mod_rel_trans_imp(a, b, c, n)
            r(a, c)
        }
    }
    is_transitive(r) = forall(x: Nat, y: Nat, z: Nat) {
        r(x, y) and r(y, z) implies r(x, z)
    }
    is_transitive(r)
}

/// Congruence modulo n is an equivalence relation.
theorem congr_mod_is_equivalence(n: Nat) {
    is_equivalence(congr_mod_rel(n))
} by {
    congr_mod_rel_reflexive(n)
    congr_mod_rel_symmetric(n)
    congr_mod_rel_transitive(n)
    reflexive_symmetric_transitive_imp_equivalence(congr_mod_rel(n))
    is_equivalence(congr_mod_rel(n))
}

// ---------------------------------------------------------------------------
// (c) Multiplicativity of congruence.
// ---------------------------------------------------------------------------

/// Congruence modulo n is preserved by multiplication: if a is congruent to
/// b and c is congruent to d modulo n, then a*c is congruent to b*d.
theorem congr_mod_mul_pairs(a: Nat, b: Nat, c: Nat, d: Nat, n: Nat) {
    a.congr_mod(b, n) and c.congr_mod(d, n) implies (a * c).congr_mod(b * d, n)
} by {
    congr_mod_mul(a, c, b, d, n)
}

/// Multiplying both sides of a congruence by a constant preserves it.
theorem congr_mod_mul_left(a: Nat, b: Nat, c: Nat, n: Nat) {
    a.congr_mod(b, n) implies (a * c).congr_mod(b * c, n)
} by {
    if a.congr_mod(b, n) {
        congr_mod_refl(c, n)
        congr_mod_mul(a, c, b, c, n)
        (a * c).congr_mod(b * c, n)
    }
}

/// Multiplying both sides of a congruence by a constant on the right
/// preserves it.
theorem congr_mod_mul_right(a: Nat, b: Nat, c: Nat, n: Nat) {
    a.congr_mod(b, n) implies (c * a).congr_mod(c * b, n)
} by {
    if a.congr_mod(b, n) {
        congr_mod_refl(c, n)
        congr_mod_mul(c, a, c, b, n)
        (c * a).congr_mod(c * b, n)
    }
}

/// Adding the same constant to both sides of a congruence preserves it.
theorem congr_mod_add_left(a: Nat, b: Nat, c: Nat, n: Nat) {
    a.congr_mod(b, n) implies (a + c).congr_mod(b + c, n)
} by {
    if a.congr_mod(b, n) {
        congr_mod_refl(c, n)
        congr_mod_add(a, c, b, c, n)
        (a + c).congr_mod(b + c, n)
    }
}

/// Adding the same constant to both sides of a congruence on the right
/// preserves it.
theorem congr_mod_add_right(a: Nat, b: Nat, c: Nat, n: Nat) {
    a.congr_mod(b, n) implies (c + a).congr_mod(c + b, n)
} by {
    if a.congr_mod(b, n) {
        congr_mod_refl(c, n)
        congr_mod_add(c, a, c, b, n)
        (c + a).congr_mod(c + b, n)
    }
}

// ---------------------------------------------------------------------------
// (d) Congruence is preserved by powers.
// ---------------------------------------------------------------------------

/// Congruence modulo n is preserved by raising to a natural-number power: if

// ---------------------------------------------------------------------------
// (e) Congruence is equality of remainders.
// ---------------------------------------------------------------------------

/// Congruence modulo n is exactly equality of the two remainders modulo n.
theorem congr_mod_iff_same_mod(a: Nat, b: Nat, n: Nat) {
    a.congr_mod(b, n) = (a.mod(n) = b.mod(n))
} by {
}
