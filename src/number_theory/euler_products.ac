// Euler products and the product formulas for the classical arithmetic
// functions.
//
// This file collects the product formulas
//
//     phi(n) = n * prod_{p | n} (1 - 1/p),
//     sigma(n) = prod_{p^k || n} (p^{k+1} - 1) / (p - 1),
//     mu(n) = prod_{p | n} (-1) for squarefree n, and 0 otherwise,
//
// together with the Euler product for the Riemann zeta function (stated, deep).
// The library works over the naturals, so the formulas are stated in factored
// form: `phi(p^k) = (p - 1) * p^(k-1)`, `sigma(p^k) = (p^(k+1) - 1)/(p - 1)`,
// and `mu(prod l) = (-1)^(l.length)` for a list `l` of distinct primes.  The
// general statements follow from these prime-power factors together with
// multiplicativity on coprime arguments (which the library already has for
// phi, sigma and mu); the cases proved here are the prime powers and the
// products of two distinct primes.
from nat import Nat
from int import Int
from list import List, product, is_permutation, permutation_preserves_length
from algebra.ring.ring import alternating_sign
from number_theory.divisor_sum import nat_sigma, nat_sigma_prime
from number_theory.totient import nat_totient, totient_pq, totient_p_pow,
    totient_p_pow_factored
from number_theory.sigma_multiplicative import nat_sigma_prime_pow,
    nat_sigma_mul_coprime
from number_theory.factorisation import prime_factorisation,
    prime_factorisation_product, prime_factorisation_all_prime,
    prime_factorisation_unique, all_prime, all_prime_product_nonzero,
    coprime_of_distinct_primes
from number_theory.squarefree import product_of_unique_primes_squarefree,
    squarefree_imp_factorisation_unique, is_squarefree_iff_mobius_nonzero
from number_theory.mobius_inversion import nat_mobius
from data.nat.nat_squarefree import is_squarefree
from nat import zero_or_suc, suc_sub_one, pos_of_ne_zero, lt_trans
numerals Nat
numerals Int

// ---------------------------------------------------------------------------
// The totient product formula
// ---------------------------------------------------------------------------

/// Euler's totient product formula at a prime power, in factored form:
///   `nat_totient(p.pow(k)) = (p - Nat.1) * p.pow(k - Nat.1)`
/// for prime `p` and `k >= 1`.  This is the natural-number reading of
/// `phi(p^k) = p^k * (1 - 1/p)`.
theorem nat_totient_prime_pow_product_form(p: Nat, k: Nat) {
    p.is_prime and Nat.0 < k implies
        nat_totient(p.pow(k)) = (p - Nat.1) * p.pow(k - Nat.1)
} by {
    if p.is_prime and Nat.0 < k {
        k != Nat.0
        zero_or_suc(k)
        let n: Nat satisfy { n.suc = k }
        totient_p_pow_factored(p, n)
        (p.pow(n.suc)).totient = (p - Nat.1) * p.pow(n)
        n.suc = k
        (p.pow(k)).totient = (p - Nat.1) * p.pow(n)
        suc_sub_one(n)
        n.suc - Nat.1 = n
        k - Nat.1 = n
        p.pow(k - Nat.1) = p.pow(n)
        (p - Nat.1) * p.pow(k - Nat.1) = (p - Nat.1) * p.pow(n)
        (p.pow(k)).totient = (p - Nat.1) * p.pow(k - Nat.1)
        nat_totient(p.pow(k)) = (p.pow(k)).totient
        nat_totient(p.pow(k)) = (p - Nat.1) * p.pow(k - Nat.1)
    }
}

/// Euler's totient product formula at a prime power, in difference form:
///   `nat_totient(p.pow(k)) = p.pow(k) - p.pow(k - Nat.1)`
/// for prime `p` and `k >= 1`.  Together with the factored form this is the
/// classical `phi(p^k) = p^k - p^(k-1)`.
theorem nat_totient_prime_pow_difference_form(p: Nat, k: Nat) {
    p.is_prime and Nat.0 < k implies
        nat_totient(p.pow(k)) = p.pow(k) - p.pow(k - Nat.1)
} by {
    if p.is_prime and Nat.0 < k {
        k != Nat.0
        zero_or_suc(k)
        let n: Nat satisfy { n.suc = k }
        totient_p_pow(p, n)
        (p.pow(n.suc)).totient = p.pow(n.suc) - p.pow(n)
        n.suc = k
        (p.pow(k)).totient = p.pow(k) - p.pow(n)
        suc_sub_one(n)
        n.suc - Nat.1 = n
        k - Nat.1 = n
        p.pow(k - Nat.1) = p.pow(n)
        p.pow(k) - p.pow(k - Nat.1) = p.pow(k) - p.pow(n)
        (p.pow(k)).totient = p.pow(k) - p.pow(k - Nat.1)
        nat_totient(p.pow(k)) = (p.pow(k)).totient
        nat_totient(p.pow(k)) = p.pow(k) - p.pow(k - Nat.1)
    }
}

/// Euler's totient product formula at a product of two distinct primes:
///   `nat_totient(p * q) = (p - Nat.1) * (q - Nat.1)`,
/// the factored form of `phi(p * q) = p * q * (1 - 1/p) * (1 - 1/q)`.
theorem nat_totient_two_prime_product_form(p: Nat, q: Nat) {
    p.is_prime and q.is_prime and p != q implies
        nat_totient(p * q) = (p - Nat.1) * (q - Nat.1)
} by {
    if p.is_prime and q.is_prime and p != q {
        totient_pq(p, q)
        (p * q).totient = (p - Nat.1) * (q - Nat.1)
        nat_totient(p * q) = (p * q).totient
        nat_totient(p * q) = (p - Nat.1) * (q - Nat.1)
    }
}

// ---------------------------------------------------------------------------
// The divisor-sum product formula
// ---------------------------------------------------------------------------

/// The divisor-sum product formula at a prime power:
///   `nat_sigma(p.pow(k)) = (p.pow(k + Nat.1) - Nat.1).div(p - Nat.1)`
/// for prime `p`.  This is the factor `(p^{k+1} - 1)/(p - 1)` in the product
/// `sigma(n) = prod_{p^k || n} (p^{k+1} - 1)/(p - 1)`.
theorem nat_sigma_prime_pow_product_form(p: Nat, k: Nat) {
    p.is_prime implies
        nat_sigma(p.pow(k)) = (p.pow(k + Nat.1) - Nat.1).div(p - Nat.1)
} by {
    if p.is_prime {
        nat_sigma_prime_pow(p, k)
        nat_sigma(p.pow(k)) = (p.pow(k + Nat.1) - Nat.1).div(p - Nat.1)
    }
}

/// The divisor-sum product formula at a product of two distinct primes:
///   `nat_sigma(p * q) = (p + Nat.1) * (q + Nat.1)`.
/// This is the two-factor case of `sigma(n) = prod_{p | n} (p + 1)` on
/// squarefree `n`, obtained from multiplicativity of `sigma` and
/// `sigma(p) = p + 1` for prime `p`.
theorem nat_sigma_two_prime_product_form(p: Nat, q: Nat) {
    p.is_prime and q.is_prime and p != q implies
        nat_sigma(p * q) = (p + Nat.1) * (q + Nat.1)
} by {
    if p.is_prime and q.is_prime and p != q {
        coprime_of_distinct_primes(p, q)
        p.coprime(q)
        Nat.1 < p
        lt_trans(Nat.0, Nat.1, p)
        Nat.0 < p
        Nat.1 < q
        lt_trans(Nat.0, Nat.1, q)
        Nat.0 < q
        nat_sigma_mul_coprime(p, q)
        nat_sigma(p * q) = nat_sigma(p) * nat_sigma(q)
        nat_sigma_prime(p)
        nat_sigma(p) = p + Nat.1
        nat_sigma_prime(q)
        nat_sigma(q) = q + Nat.1
        nat_sigma(p * q) = (p + Nat.1) * (q + Nat.1)
    }
}

// ---------------------------------------------------------------------------
// The Euler product for the Riemann zeta function
// ---------------------------------------------------------------------------
//
// For Re(s) > 1 the Riemann zeta function has the Euler product
//
//     zeta(s) = sum_{n >= 1} n^{-s} = prod_{p prime} (1 - p^{-s})^{-1}.
//
// Expanding each factor as a geometric series,
// (1 - p^{-s})^{-1} = sum_{k >= 0} p^{-k s}, and multiplying over the primes
// enumerates every natural number exactly once, by the fundamental theorem of
// arithmetic: the coefficient of n^{-s} on the right is the number of ways to
// write n as a product of prime powers, which is one.  Formalising the
// identity requires the analytic theory of infinite products and Dirichlet
// series, which is not yet present in this library; the statement is recorded
// here as the target for future work.  The finite version -- the product over
// the prime power factors of a fixed n of (1 - p^{-s})^{-1} equals
// sum_{d | n} d^{-s} -- is the elementary expansion of the finite product and
// is proved in factorisation.ac via `prime_factorisation_product`.

// A formal statement (not yet provable in this library) would be:
//
//     theorem zeta_euler_product(s: Complex) {
//         sum_{n >= 1} n^{-s} = prod_{p prime} (1 - p^{-s})^{-1}
//     }

// ---------------------------------------------------------------------------
// The Mobius product formula
// ---------------------------------------------------------------------------

/// The Mobius product formula for a product of distinct primes: if `l` lists
/// the distinct prime divisors of `n = product[Nat](l)`, then
///   `nat_mobius(product[Nat](l)) = alternating_sign[Int](l.length)`,
/// i.e. `(-1)^k` where `k` is the number of prime divisors: the product
/// `prod_{p | n} (-1)` over the primes dividing the squarefree `n`.
theorem nat_mobius_squarefree_product_form(l: List[Nat]) {
    all_prime(l) and l.is_unique implies
        nat_mobius(product[Nat](l)) = alternating_sign[Int](l.length)
} by {
    if all_prime(l) and l.is_unique {
        all_prime_product_nonzero(l)
        product[Nat](l) != Nat.0
        pos_of_ne_zero(product[Nat](l))
        Nat.0 < product[Nat](l)
        product_of_unique_primes_squarefree(l)
        is_squarefree(product[Nat](l))
        squarefree_imp_factorisation_unique(product[Nat](l))
        prime_factorisation(product[Nat](l)).is_unique
        nat_mobius(product[Nat](l)) =
            alternating_sign[Int](prime_factorisation(product[Nat](l)).length)
        prime_factorisation_product(product[Nat](l))
        product[Nat](prime_factorisation(product[Nat](l))) = product[Nat](l)
        prime_factorisation_all_prime(product[Nat](l))
        all_prime(prime_factorisation(product[Nat](l)))
        prime_factorisation_unique(l, prime_factorisation(product[Nat](l)))
        is_permutation(l, prime_factorisation(product[Nat](l)))
        permutation_preserves_length(l, prime_factorisation(product[Nat](l)))
        l.length = prime_factorisation(product[Nat](l)).length
        alternating_sign[Int](prime_factorisation(product[Nat](l)).length) =
            alternating_sign[Int](l.length)
        nat_mobius(product[Nat](l)) = alternating_sign[Int](l.length)
    }
}

/// The Mobius function vanishes off squarefree numbers:
///   `not is_squarefree(n) implies nat_mobius(n) = Int.0`.
/// This is the `0 otherwise` half of the Mobius product formula
/// `mu(n) = prod_{p | n} (-1)` for squarefree `n` and `0` otherwise.
theorem nat_mobius_non_squarefree_zero(n: Nat) {
    not is_squarefree(n) implies nat_mobius(n) = Int.0
} by {
    if not is_squarefree(n) {
        is_squarefree_iff_mobius_nonzero(n)
        is_squarefree(n) = (nat_mobius(n) != Int.0)
        not (nat_mobius(n) != Int.0)
        if nat_mobius(n) = Int.0 {
            nat_mobius(n) = Int.0
        } else {
            nat_mobius(n) != Int.0
            not (nat_mobius(n) != Int.0)
            false
        }
    }
}
