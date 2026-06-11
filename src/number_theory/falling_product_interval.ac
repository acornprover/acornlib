from nat import Nat, add_cancels_left, double_addition_carry_count, digit_sum
from number_theory.factorisation import count_prime_factor
from number_theory.falling_product import central_binom, falling_product,
    falling_product_prime_count_sum, count_prime_factor_falling_product
from number_theory.falling_product_binomial import complement_upto_add_falling_product_prime_count_sum,
    falling_product_prime_count_sum_add_complement_upto
from number_theory.falling_product_divisibility import falling_product_target_prime_bound,
    falling_product_target_primewise_bound,
    count_prime_factor_falling_product_le_target_of_target_prime_bound,
    count_prime_factor_falling_product_le_target_of_target_primewise_bound,
    falling_product_divides_target_of_target_primewise_bound,
    falling_product_divides_target_iff_target_primewise_bound,
    falling_product_target_primewise_bound_of_divides,
    falling_product_central_binom_prime_bound,
    falling_product_central_binom_carry_bound,
    falling_product_central_binom_binary_digit_bound,
    falling_product_central_binom_carry_bound_of_prime_bound,
    falling_product_central_binom_prime_bound_of_carry_bound,
    falling_product_central_binom_carry_bound_two_of_binary_digit_bound,
    falling_product_central_binom_binary_digit_bound_of_carry_bound_two,
    falling_product_central_binom_primewise_bound,
    falling_product_central_binom_carrywise_bound,
    count_prime_factor_falling_product_le_central_binom_of_prime_bound,
    count_prime_factor_falling_product_le_central_binom_of_primewise_bound,
    falling_product_divides_central_binom_of_primewise_bound,
    falling_product_divides_central_binom_of_carrywise_bound,
    falling_product_central_binom_primewise_bound_of_divides,
    falling_product_central_binom_carrywise_bound_of_divides,
    falling_product_divides_central_binom_iff_primewise_bound,
    falling_product_divides_central_binom_iff_carrywise_bound
from number_theory.legendre import prime_factor_count_upto

numerals Nat

/// True when `value` is the valuation sum over the interval from `lower` to
/// `upper`, expressed by the difference of Legendre sums.
define prime_factor_count_interval_witness(
    p: Nat, lower: Nat, upper: Nat, value: Nat
) -> Bool {
    prime_factor_count_upto(p, lower) + value = prime_factor_count_upto(p, upper)
}

/// True when an interval valuation sum is bounded by a given natural.
define prime_factor_count_interval_bound(
    p: Nat, lower: Nat, upper: Nat, value: Nat, bound: Nat
) -> Bool {
    prime_factor_count_interval_witness(p, lower, upper, value) and value <= bound
}

/// True when an interval valuation sum is bounded by a target valuation.
define prime_factor_count_interval_target_bound(
    p: Nat, lower: Nat, upper: Nat, value: Nat, target: Nat
) -> Bool {
    prime_factor_count_interval_bound(p, lower, upper, value,
        count_prime_factor(p, target))
}

/// True when an interval valuation sum is bounded by the central binomial
/// valuation.
define prime_factor_count_interval_central_binom_prime_bound(
    p: Nat, n: Nat, lower: Nat, value: Nat
) -> Bool {
    prime_factor_count_interval_target_bound(p, lower, n, value, central_binom(n))
}

/// True when an interval valuation sum is bounded by the central binomial
/// carry count.
define prime_factor_count_interval_central_binom_carry_bound(
    p: Nat, n: Nat, lower: Nat, value: Nat
) -> Bool {
    prime_factor_count_interval_bound(p, lower, n, value,
        double_addition_carry_count(p, n))
}

/// True when a binary interval valuation sum is bounded by the binary digit
/// sum.
define prime_factor_count_interval_central_binom_binary_digit_bound(
    n: Nat, lower: Nat, value: Nat
) -> Bool {
    prime_factor_count_interval_bound(Nat.2, lower, n, value, digit_sum(Nat.2, n))
}

/// The lower endpoint for the falling-product interval `n - k, ..., n`.
define falling_product_interval_lower(n: Nat, k: Nat) -> Nat {
    n - k.suc
}

/// The valuation carried by the falling-product interval at a prime.
define falling_product_interval_value(p: Nat, n: Nat, k: Nat) -> Nat {
    falling_product_prime_count_sum(p, n, k)
}

/// True when `value` is the interval valuation sum represented by a falling
/// product.
define falling_product_interval_witness(
    p: Nat, n: Nat, k: Nat, value: Nat
) -> Bool {
    prime_factor_count_interval_witness(p, falling_product_interval_lower(n, k),
        n, value)
}

/// True when a falling-product interval valuation sum is bounded by a target
/// valuation.
define falling_product_interval_target_bound(
    p: Nat, n: Nat, k: Nat, value: Nat, target: Nat
) -> Bool {
    prime_factor_count_interval_target_bound(p, falling_product_interval_lower(n, k),
        n, value, target)
}

/// True when every prime falling-product interval valuation is bounded by the
/// matching target valuation.
define falling_product_interval_target_primewise_bound(
    n: Nat, k: Nat, target: Nat
) -> Bool {
    forall(p: Nat) {
        p.is_prime implies falling_product_interval_target_bound(
            p, n, k, falling_product_interval_value(p, n, k), target)
    }
}

/// True when a falling-product interval valuation is bounded by the central
/// binomial valuation.
define falling_product_interval_central_binom_prime_bound(
    p: Nat, n: Nat, k: Nat, value: Nat
) -> Bool {
    prime_factor_count_interval_central_binom_prime_bound(p, n,
        falling_product_interval_lower(n, k), value)
}

/// True when a falling-product interval valuation is bounded by the central
/// binomial carry count.
define falling_product_interval_central_binom_carry_bound(
    p: Nat, n: Nat, k: Nat, value: Nat
) -> Bool {
    prime_factor_count_interval_central_binom_carry_bound(p, n,
        falling_product_interval_lower(n, k), value)
}

/// True when a binary falling-product interval valuation is bounded by the
/// binary digit sum.
define falling_product_interval_central_binom_binary_digit_bound(
    n: Nat, k: Nat, value: Nat
) -> Bool {
    prime_factor_count_interval_central_binom_binary_digit_bound(n,
        falling_product_interval_lower(n, k), value)
}

/// True when every prime falling-product interval valuation is bounded by the
/// matching central binomial valuation.
define falling_product_interval_central_binom_primewise_bound(n: Nat, k: Nat) -> Bool {
    forall(p: Nat) {
        p.is_prime implies falling_product_interval_central_binom_prime_bound(
            p, n, k, falling_product_interval_value(p, n, k))
    }
}

/// True when every prime falling-product interval valuation is bounded by the
/// matching central binomial carry count.
define falling_product_interval_central_binom_carrywise_bound(n: Nat, k: Nat) -> Bool {
    forall(p: Nat) {
        p.is_prime implies falling_product_interval_central_binom_carry_bound(
            p, n, k, falling_product_interval_value(p, n, k))
    }
}

/// The same endpoint interval has valuation zero.
theorem prime_factor_count_interval_witness_zero(p: Nat, n: Nat) {
    prime_factor_count_interval_witness(p, n, n, Nat.0)
} by {
    prime_factor_count_upto(p, n) + Nat.0 = prime_factor_count_upto(p, n)
    prime_factor_count_interval_witness(p, n, n, Nat.0)
}

/// An interval valuation witness is unique.
theorem prime_factor_count_interval_witness_unique(
    p: Nat, lower: Nat, upper: Nat, a: Nat, b: Nat
) {
    prime_factor_count_interval_witness(p, lower, upper, a) and
    prime_factor_count_interval_witness(p, lower, upper, b) implies a = b
} by {
    if prime_factor_count_interval_witness(p, lower, upper, a) and
        prime_factor_count_interval_witness(p, lower, upper, b) {
        prime_factor_count_upto(p, lower) + a = prime_factor_count_upto(p, upper)
        prime_factor_count_upto(p, lower) + b = prime_factor_count_upto(p, upper)
        prime_factor_count_upto(p, lower) + a = prime_factor_count_upto(p, lower) + b
        add_cancels_left(prime_factor_count_upto(p, lower), a, b)
        a = b
    }
}

/// An interval valuation witness bounds the value by the upper Legendre sum.
theorem prime_factor_count_interval_witness_value_le_upper(
    p: Nat, lower: Nat, upper: Nat, value: Nat
) {
    prime_factor_count_interval_witness(p, lower, upper, value)
        implies value <= prime_factor_count_upto(p, upper)
} by {
    if prime_factor_count_interval_witness(p, lower, upper, value) {
        let lower_count: Nat = prime_factor_count_upto(p, lower)
        let upper_count: Nat = prime_factor_count_upto(p, upper)
        lower_count + value = upper_count
        value + lower_count = lower_count + value
        value + lower_count = upper_count
        value <= upper_count
        value <= prime_factor_count_upto(p, upper)
    }
}

/// An interval valuation witness bounds the lower Legendre sum by the upper
/// Legendre sum.
theorem prime_factor_count_interval_witness_lower_le_upper(
    p: Nat, lower: Nat, upper: Nat, value: Nat
) {
    prime_factor_count_interval_witness(p, lower, upper, value)
        implies prime_factor_count_upto(p, lower) <= prime_factor_count_upto(p, upper)
} by {
    if prime_factor_count_interval_witness(p, lower, upper, value) {
        let lower_count: Nat = prime_factor_count_upto(p, lower)
        let upper_count: Nat = prime_factor_count_upto(p, upper)
        lower_count + value = upper_count
        lower_count <= upper_count
        prime_factor_count_upto(p, lower) <= prime_factor_count_upto(p, upper)
    }
}

/// A bounded interval witness has a valuation witness.
theorem prime_factor_count_interval_witness_of_bound(
    p: Nat, lower: Nat, upper: Nat, value: Nat, bound: Nat
) {
    prime_factor_count_interval_bound(p, lower, upper, value, bound)
        implies prime_factor_count_interval_witness(p, lower, upper, value)
} by {
    if prime_factor_count_interval_bound(p, lower, upper, value, bound) {
        prime_factor_count_interval_witness(p, lower, upper, value)
    }
}

/// A bounded interval witness has the stated bound.
theorem prime_factor_count_interval_value_le_of_bound(
    p: Nat, lower: Nat, upper: Nat, value: Nat, bound: Nat
) {
    prime_factor_count_interval_bound(p, lower, upper, value, bound)
        implies value <= bound
} by {
    if prime_factor_count_interval_bound(p, lower, upper, value, bound) {
        value <= bound
    }
}

/// A target-bounded interval witness has a valuation witness.
theorem prime_factor_count_interval_witness_of_target_bound(
    p: Nat, lower: Nat, upper: Nat, value: Nat, target: Nat
) {
    prime_factor_count_interval_target_bound(p, lower, upper, value, target)
        implies prime_factor_count_interval_witness(p, lower, upper, value)
} by {
    if prime_factor_count_interval_target_bound(p, lower, upper, value, target) {
        prime_factor_count_interval_bound(p, lower, upper, value,
            count_prime_factor(p, target))
        prime_factor_count_interval_witness(p, lower, upper, value)
    }
}

/// A target-bounded interval witness has the target valuation bound.
theorem prime_factor_count_interval_value_le_target_of_target_bound(
    p: Nat, lower: Nat, upper: Nat, value: Nat, target: Nat
) {
    prime_factor_count_interval_target_bound(p, lower, upper, value, target)
        implies value <= count_prime_factor(p, target)
} by {
    if prime_factor_count_interval_target_bound(p, lower, upper, value, target) {
        prime_factor_count_interval_bound(p, lower, upper, value,
            count_prime_factor(p, target))
        value <= count_prime_factor(p, target)
    }
}

/// The falling-product interval value is the falling-product valuation sum.
theorem falling_product_interval_value_eq_prime_count_sum(p: Nat, n: Nat, k: Nat) {
    falling_product_interval_value(p, n, k) = falling_product_prime_count_sum(p, n, k)
}

/// The falling-product valuation sum is an interval valuation witness.
theorem falling_product_prime_count_sum_interval_witness(
    p: Nat, n: Nat, k: Nat
) {
    k < n implies falling_product_interval_witness(
        p, n, k, falling_product_prime_count_sum(p, n, k))
} by {
    if k < n {
        complement_upto_add_falling_product_prime_count_sum(p, n, k)
        prime_factor_count_upto(p, n - k.suc) +
            falling_product_prime_count_sum(p, n, k) =
                prime_factor_count_upto(p, n)
        falling_product_interval_lower(n, k) = n - k.suc
        prime_factor_count_interval_witness(p, falling_product_interval_lower(n, k),
            n, falling_product_prime_count_sum(p, n, k))
        falling_product_interval_witness(
            p, n, k, falling_product_prime_count_sum(p, n, k))
    }
}

/// The canonical falling-product interval value is an interval valuation
/// witness.
theorem falling_product_interval_value_witness(p: Nat, n: Nat, k: Nat) {
    k < n implies falling_product_interval_witness(
        p, n, k, falling_product_interval_value(p, n, k))
} by {
    if k < n {
        falling_product_interval_value_eq_prime_count_sum(p, n, k)
        falling_product_interval_value(p, n, k) = falling_product_prime_count_sum(p, n, k)
        falling_product_prime_count_sum_interval_witness(p, n, k)
        falling_product_interval_witness(
            p, n, k, falling_product_prime_count_sum(p, n, k))
        falling_product_interval_witness(
            p, n, k, falling_product_interval_value(p, n, k))
    }
}

/// The valuation of a positive falling product is an interval valuation
/// witness.
theorem count_prime_factor_falling_product_interval_witness(
    p: Nat, n: Nat, k: Nat
) {
    k < n implies falling_product_interval_witness(
        p, n, k, count_prime_factor(p, falling_product(n, k)))
} by {
    if k < n {
        count_prime_factor_falling_product(p, n, k)
        count_prime_factor(p, falling_product(n, k)) =
            falling_product_prime_count_sum(p, n, k)
        falling_product_prime_count_sum_interval_witness(p, n, k)
        falling_product_interval_witness(
            p, n, k, falling_product_prime_count_sum(p, n, k))
        falling_product_interval_witness(
            p, n, k, count_prime_factor(p, falling_product(n, k)))
    }
}

/// Any falling-product interval witness has the canonical interval value.
theorem falling_product_interval_witness_value_eq(
    p: Nat, n: Nat, k: Nat, value: Nat
) {
    k < n and falling_product_interval_witness(p, n, k, value)
        implies value = falling_product_interval_value(p, n, k)
} by {
    if k < n and falling_product_interval_witness(p, n, k, value) {
        falling_product_interval_value_witness(p, n, k)
        falling_product_interval_witness(
            p, n, k, falling_product_interval_value(p, n, k))
        prime_factor_count_interval_witness(p, falling_product_interval_lower(n, k),
            n, value)
        prime_factor_count_interval_witness(p, falling_product_interval_lower(n, k),
            n, falling_product_interval_value(p, n, k))
        prime_factor_count_interval_witness_unique(p,
            falling_product_interval_lower(n, k), n, value,
            falling_product_interval_value(p, n, k))
        value = falling_product_interval_value(p, n, k)
    }
}

/// Any falling-product interval witness has the falling-product valuation sum.
theorem falling_product_interval_witness_value_eq_prime_count_sum(
    p: Nat, n: Nat, k: Nat, value: Nat
) {
    k < n and falling_product_interval_witness(p, n, k, value)
        implies value = falling_product_prime_count_sum(p, n, k)
} by {
    if k < n and falling_product_interval_witness(p, n, k, value) {
        falling_product_interval_witness_value_eq(p, n, k, value)
        value = falling_product_interval_value(p, n, k)
        falling_product_interval_value_eq_prime_count_sum(p, n, k)
        falling_product_interval_value(p, n, k) =
            falling_product_prime_count_sum(p, n, k)
        value = falling_product_prime_count_sum(p, n, k)
    }
}

/// A falling-product target valuation bound gives the corresponding interval
/// target bound.
theorem falling_product_interval_target_bound_of_target_prime_bound(
    p: Nat, n: Nat, k: Nat, target: Nat
) {
    k < n and falling_product_target_prime_bound(p, n, k, target)
        implies falling_product_interval_target_bound(
            p, n, k, falling_product_interval_value(p, n, k), target)
} by {
    if k < n and falling_product_target_prime_bound(p, n, k, target) {
        falling_product_interval_value_witness(p, n, k)
        falling_product_interval_witness(
            p, n, k, falling_product_interval_value(p, n, k))
        falling_product_prime_count_sum(p, n, k) <= count_prime_factor(p, target)
        falling_product_interval_value_eq_prime_count_sum(p, n, k)
        falling_product_interval_value(p, n, k) =
            falling_product_prime_count_sum(p, n, k)
        falling_product_interval_value(p, n, k) <= count_prime_factor(p, target)
        prime_factor_count_interval_bound(p, falling_product_interval_lower(n, k),
            n, falling_product_interval_value(p, n, k), count_prime_factor(p, target))
        falling_product_interval_target_bound(
            p, n, k, falling_product_interval_value(p, n, k), target)
    }
}

/// A falling-product interval target bound gives the corresponding target
/// valuation bound.
theorem falling_product_target_prime_bound_of_interval_target_bound(
    p: Nat, n: Nat, k: Nat, value: Nat, target: Nat
) {
    k < n and falling_product_interval_target_bound(p, n, k, value, target)
        implies falling_product_target_prime_bound(p, n, k, target)
} by {
    if k < n and falling_product_interval_target_bound(p, n, k, value, target) {
        prime_factor_count_interval_target_bound(p, falling_product_interval_lower(n, k),
            n, value, target)
        prime_factor_count_interval_witness_of_target_bound(p,
            falling_product_interval_lower(n, k), n, value, target)
        prime_factor_count_interval_witness(p, falling_product_interval_lower(n, k),
            n, value)
        falling_product_interval_witness(p, n, k, value)
        falling_product_interval_witness_value_eq_prime_count_sum(p, n, k, value)
        value = falling_product_prime_count_sum(p, n, k)
        prime_factor_count_interval_value_le_target_of_target_bound(p,
            falling_product_interval_lower(n, k), n, value, target)
        value <= count_prime_factor(p, target)
        falling_product_prime_count_sum(p, n, k) <= count_prime_factor(p, target)
        falling_product_target_prime_bound(p, n, k, target)
    }
}

/// A falling-product primewise target bound gives the interval target bounds.
theorem falling_product_interval_target_primewise_bound_of_target_primewise_bound(
    n: Nat, k: Nat, target: Nat
) {
    k < n and falling_product_target_primewise_bound(n, k, target)
        implies falling_product_interval_target_primewise_bound(n, k, target)
} by {
    if k < n and falling_product_target_primewise_bound(n, k, target) {
        forall(p: Nat) {
            if p.is_prime {
                let h: Bool = p.is_prime implies falling_product_target_prime_bound(
                    p, n, k, target)
                h
                falling_product_target_prime_bound(p, n, k, target)
                falling_product_interval_target_bound_of_target_prime_bound(
                    p, n, k, target)
                falling_product_interval_target_bound(
                    p, n, k, falling_product_interval_value(p, n, k), target)
            }
        }
        falling_product_interval_target_primewise_bound(n, k, target)
    }
}

/// Falling-product interval target bounds give the primewise target bound.
theorem falling_product_target_primewise_bound_of_interval_target_primewise_bound(
    n: Nat, k: Nat, target: Nat
) {
    k < n and falling_product_interval_target_primewise_bound(n, k, target)
        implies falling_product_target_primewise_bound(n, k, target)
} by {
    if k < n and falling_product_interval_target_primewise_bound(n, k, target) {
        forall(p: Nat) {
            if p.is_prime {
                let h: Bool = p.is_prime implies falling_product_interval_target_bound(
                    p, n, k, falling_product_interval_value(p, n, k), target)
                h
                falling_product_interval_target_bound(
                    p, n, k, falling_product_interval_value(p, n, k), target)
                falling_product_target_prime_bound_of_interval_target_bound(
                    p, n, k, falling_product_interval_value(p, n, k), target)
                falling_product_target_prime_bound(p, n, k, target)
            }
        }
        falling_product_target_primewise_bound(n, k, target)
    }
}

/// The interval target bounds are equivalent to the falling-product target
/// valuation bounds.
theorem falling_product_interval_target_primewise_bound_iff_target_primewise_bound(
    n: Nat, k: Nat, target: Nat
) {
    k < n implies (
        falling_product_interval_target_primewise_bound(n, k, target) =
        falling_product_target_primewise_bound(n, k, target)
    )
} by {
    if k < n {
        if falling_product_interval_target_primewise_bound(n, k, target) {
            falling_product_target_primewise_bound_of_interval_target_primewise_bound(
                n, k, target)
        }
        if falling_product_target_primewise_bound(n, k, target) {
            falling_product_interval_target_primewise_bound_of_target_primewise_bound(
                n, k, target)
        }
    }
}

/// Interval target bounds imply divisibility of the target by the falling
/// product.
theorem falling_product_divides_target_of_interval_target_primewise_bound(
    n: Nat, k: Nat, target: Nat
) {
    k < n and target != Nat.0 and
    falling_product_interval_target_primewise_bound(n, k, target)
        implies falling_product(n, k).divides(target)
} by {
    if k < n and target != Nat.0 and
        falling_product_interval_target_primewise_bound(n, k, target) {
        falling_product_target_primewise_bound_of_interval_target_primewise_bound(
            n, k, target)
        falling_product_target_primewise_bound(n, k, target)
        falling_product_divides_target_of_target_primewise_bound(n, k, target)
        falling_product(n, k).divides(target)
    }
}

/// Falling-product divisibility of a target gives the interval target bounds.
theorem falling_product_interval_target_primewise_bound_of_divides(
    n: Nat, k: Nat, target: Nat
) {
    k < n and target != Nat.0 and falling_product(n, k).divides(target)
        implies falling_product_interval_target_primewise_bound(n, k, target)
} by {
    if k < n and target != Nat.0 and falling_product(n, k).divides(target) {
        falling_product_target_primewise_bound_of_divides(n, k, target)
        falling_product_target_primewise_bound(n, k, target)
        falling_product_interval_target_primewise_bound_of_target_primewise_bound(
            n, k, target)
        falling_product_interval_target_primewise_bound(n, k, target)
    }
}

/// A positive falling product divides a target iff all interval target bounds
/// hold.
theorem falling_product_divides_target_iff_interval_target_primewise_bound(
    n: Nat, k: Nat, target: Nat
) {
    k < n and target != Nat.0 implies (
        falling_product(n, k).divides(target) =
        falling_product_interval_target_primewise_bound(n, k, target)
    )
} by {
    if k < n and target != Nat.0 {
        let divides_target: Bool = falling_product(n, k).divides(target)
        let target_bound: Bool = falling_product_target_primewise_bound(n, k, target)
        let interval_bound: Bool =
            falling_product_interval_target_primewise_bound(n, k, target)
        falling_product_divides_target_iff_target_primewise_bound(n, k, target)
        divides_target = target_bound
        falling_product_interval_target_primewise_bound_iff_target_primewise_bound(
            n, k, target)
        interval_bound = target_bound
        target_bound = interval_bound
        divides_target = interval_bound
        falling_product(n, k).divides(target) =
            falling_product_interval_target_primewise_bound(n, k, target)
    }
}

/// The canonical interval target bound is equivalent to the existing
/// falling-product target valuation bound.
theorem falling_product_interval_target_bound_iff_target_prime_bound(
    p: Nat, n: Nat, k: Nat, target: Nat
) {
    k < n implies (
        falling_product_interval_target_bound(
            p, n, k, falling_product_interval_value(p, n, k), target) =
        falling_product_target_prime_bound(p, n, k, target)
    )
} by {
    if k < n {
        if falling_product_interval_target_bound(
            p, n, k, falling_product_interval_value(p, n, k), target) {
            falling_product_target_prime_bound_of_interval_target_bound(
                p, n, k, falling_product_interval_value(p, n, k), target)
        }
        if falling_product_target_prime_bound(p, n, k, target) {
            falling_product_interval_target_bound_of_target_prime_bound(
                p, n, k, target)
        }
    }
}

/// A primewise target valuation bound gives the interval target bound at a
/// particular prime.
theorem falling_product_interval_target_bound_of_target_primewise_bound(
    p: Nat, n: Nat, k: Nat, target: Nat
) {
    p.is_prime and k < n and falling_product_target_primewise_bound(n, k, target)
        implies falling_product_interval_target_bound(
            p, n, k, falling_product_interval_value(p, n, k), target)
} by {
    if p.is_prime and k < n and falling_product_target_primewise_bound(n, k, target) {
        let h: Bool = p.is_prime implies falling_product_target_prime_bound(
            p, n, k, target)
        h
        falling_product_target_prime_bound(p, n, k, target)
        falling_product_interval_target_bound_of_target_prime_bound(p, n, k, target)
        falling_product_interval_target_bound(
            p, n, k, falling_product_interval_value(p, n, k), target)
    }
}

/// An interval primewise target bound gives the interval target bound at a
/// particular prime.
theorem falling_product_interval_target_bound_of_interval_target_primewise_bound(
    p: Nat, n: Nat, k: Nat, target: Nat
) {
    p.is_prime and falling_product_interval_target_primewise_bound(n, k, target)
        implies falling_product_interval_target_bound(
            p, n, k, falling_product_interval_value(p, n, k), target)
} by {
    if p.is_prime and falling_product_interval_target_primewise_bound(n, k, target) {
        let h: Bool = p.is_prime implies falling_product_interval_target_bound(
            p, n, k, falling_product_interval_value(p, n, k), target)
        h
        falling_product_interval_target_bound(
            p, n, k, falling_product_interval_value(p, n, k), target)
    }
}

/// An interval primewise target bound gives the existing target valuation
/// bound at a particular prime.
theorem falling_product_target_prime_bound_of_interval_target_primewise_bound(
    p: Nat, n: Nat, k: Nat, target: Nat
) {
    p.is_prime and k < n and
    falling_product_interval_target_primewise_bound(n, k, target)
        implies falling_product_target_prime_bound(p, n, k, target)
} by {
    if p.is_prime and k < n and
        falling_product_interval_target_primewise_bound(n, k, target) {
        falling_product_interval_target_bound_of_interval_target_primewise_bound(
            p, n, k, target)
        falling_product_interval_target_bound(
            p, n, k, falling_product_interval_value(p, n, k), target)
        falling_product_target_prime_bound_of_interval_target_bound(
            p, n, k, falling_product_interval_value(p, n, k), target)
        falling_product_target_prime_bound(p, n, k, target)
    }
}

/// An interval target bound gives the corresponding prime-count inequality
/// for the falling product.
theorem count_prime_factor_falling_product_le_target_of_interval_target_bound(
    p: Nat, n: Nat, k: Nat, value: Nat, target: Nat
) {
    k < n and falling_product_interval_target_bound(p, n, k, value, target)
        implies count_prime_factor(p, falling_product(n, k)) <= count_prime_factor(p, target)
} by {
    if k < n and falling_product_interval_target_bound(p, n, k, value, target) {
        falling_product_target_prime_bound_of_interval_target_bound(
            p, n, k, value, target)
        falling_product_target_prime_bound(p, n, k, target)
        count_prime_factor_falling_product_le_target_of_target_prime_bound(
            p, n, k, target)
        count_prime_factor(p, falling_product(n, k)) <= count_prime_factor(p, target)
    }
}

/// Interval primewise target bounds give the prime-count inequalities needed
/// for target divisibility.
theorem count_prime_factor_falling_product_le_target_of_interval_target_primewise_bound(
    p: Nat, n: Nat, k: Nat, target: Nat
) {
    p.is_prime and k < n and
    falling_product_interval_target_primewise_bound(n, k, target)
        implies count_prime_factor(p, falling_product(n, k)) <= count_prime_factor(p, target)
} by {
    if p.is_prime and k < n and
        falling_product_interval_target_primewise_bound(n, k, target) {
        falling_product_target_primewise_bound_of_interval_target_primewise_bound(
            n, k, target)
        falling_product_target_primewise_bound(n, k, target)
        count_prime_factor_falling_product_le_target_of_target_primewise_bound(
            p, n, k, target)
        count_prime_factor(p, falling_product(n, k)) <= count_prime_factor(p, target)
    }
}

/// Interval target bounds specialized to the central binomial coefficient give
/// the interval central binomial valuation bounds.
theorem falling_product_interval_central_binom_primewise_bound_of_target_central_binom_bound(
    n: Nat, k: Nat
) {
    falling_product_interval_target_primewise_bound(n, k, central_binom(n))
        implies falling_product_interval_central_binom_primewise_bound(n, k)
} by {
    if falling_product_interval_target_primewise_bound(n, k, central_binom(n)) {
        forall(p: Nat) {
            if p.is_prime {
                falling_product_interval_target_bound_of_interval_target_primewise_bound(
                    p, n, k, central_binom(n))
                falling_product_interval_target_bound(
                    p, n, k, falling_product_interval_value(p, n, k), central_binom(n))
                falling_product_interval_central_binom_prime_bound(
                    p, n, k, falling_product_interval_value(p, n, k))
            }
        }
        falling_product_interval_central_binom_primewise_bound(n, k)
    }
}

/// Interval central binomial valuation bounds give the interval target bounds
/// specialized to the central binomial coefficient.
theorem falling_product_interval_target_central_binom_bound_of_primewise_bound(
    n: Nat, k: Nat
) {
    falling_product_interval_central_binom_primewise_bound(n, k)
        implies falling_product_interval_target_primewise_bound(n, k, central_binom(n))
} by {
    if falling_product_interval_central_binom_primewise_bound(n, k) {
        forall(p: Nat) {
            if p.is_prime {
                let h: Bool = p.is_prime implies falling_product_interval_central_binom_prime_bound(
                    p, n, k, falling_product_interval_value(p, n, k))
                h
                falling_product_interval_central_binom_prime_bound(
                    p, n, k, falling_product_interval_value(p, n, k))
                falling_product_interval_target_bound(
                    p, n, k, falling_product_interval_value(p, n, k), central_binom(n))
            }
        }
        falling_product_interval_target_primewise_bound(n, k, central_binom(n))
    }
}

/// Interval central binomial valuation bounds are the same as interval target
/// bounds specialized to the central binomial coefficient.
theorem falling_product_interval_target_central_binom_bound_iff_primewise_bound(
    n: Nat, k: Nat
) {
    falling_product_interval_target_primewise_bound(n, k, central_binom(n)) =
        falling_product_interval_central_binom_primewise_bound(n, k)
} by {
    if falling_product_interval_target_primewise_bound(n, k, central_binom(n)) {
        falling_product_interval_central_binom_primewise_bound_of_target_central_binom_bound(
            n, k)
    }
    if falling_product_interval_central_binom_primewise_bound(n, k) {
        falling_product_interval_target_central_binom_bound_of_primewise_bound(n, k)
    }
}

/// A central binomial valuation bound gives the corresponding interval
/// central binomial valuation bound.
theorem falling_product_interval_central_binom_prime_bound_of_prime_bound(
    p: Nat, n: Nat, k: Nat
) {
    k < n and falling_product_central_binom_prime_bound(p, n, k)
        implies falling_product_interval_central_binom_prime_bound(
            p, n, k, falling_product_interval_value(p, n, k))
} by {
    if k < n and falling_product_central_binom_prime_bound(p, n, k) {
        falling_product_target_prime_bound(p, n, k, central_binom(n))
        falling_product_interval_target_bound_of_target_prime_bound(
            p, n, k, central_binom(n))
        falling_product_interval_target_bound(
            p, n, k, falling_product_interval_value(p, n, k), central_binom(n))
        falling_product_interval_central_binom_prime_bound(
            p, n, k, falling_product_interval_value(p, n, k))
    }
}

/// An interval central binomial valuation bound gives the corresponding
/// falling-product valuation bound.
theorem falling_product_central_binom_prime_bound_of_interval_prime_bound(
    p: Nat, n: Nat, k: Nat, value: Nat
) {
    k < n and falling_product_interval_central_binom_prime_bound(p, n, k, value)
        implies falling_product_central_binom_prime_bound(p, n, k)
} by {
    if k < n and falling_product_interval_central_binom_prime_bound(p, n, k, value) {
        falling_product_interval_target_bound(p, n, k, value, central_binom(n))
        falling_product_target_prime_bound_of_interval_target_bound(
            p, n, k, value, central_binom(n))
        falling_product_target_prime_bound(p, n, k, central_binom(n))
        falling_product_central_binom_prime_bound(p, n, k)
    }
}

/// The canonical interval central binomial valuation bound is equivalent to
/// the existing falling-product valuation bound.
theorem falling_product_interval_central_binom_prime_bound_iff_prime_bound(
    p: Nat, n: Nat, k: Nat
) {
    k < n implies (
        falling_product_interval_central_binom_prime_bound(
            p, n, k, falling_product_interval_value(p, n, k)) =
        falling_product_central_binom_prime_bound(p, n, k)
    )
} by {
    if k < n {
        if falling_product_interval_central_binom_prime_bound(
            p, n, k, falling_product_interval_value(p, n, k)) {
            falling_product_central_binom_prime_bound_of_interval_prime_bound(
                p, n, k, falling_product_interval_value(p, n, k))
        }
        if falling_product_central_binom_prime_bound(p, n, k) {
            falling_product_interval_central_binom_prime_bound_of_prime_bound(p, n, k)
        }
    }
}

/// A central binomial carry-count bound gives the corresponding interval
/// carry-count bound.
theorem falling_product_interval_central_binom_carry_bound_of_carry_bound(
    p: Nat, n: Nat, k: Nat
) {
    k < n and falling_product_central_binom_carry_bound(p, n, k)
        implies falling_product_interval_central_binom_carry_bound(
            p, n, k, falling_product_interval_value(p, n, k))
} by {
    if k < n and falling_product_central_binom_carry_bound(p, n, k) {
        falling_product_interval_value_witness(p, n, k)
        falling_product_interval_witness(
            p, n, k, falling_product_interval_value(p, n, k))
        falling_product_prime_count_sum(p, n, k) <= double_addition_carry_count(p, n)
        falling_product_interval_value_eq_prime_count_sum(p, n, k)
        falling_product_interval_value(p, n, k) =
            falling_product_prime_count_sum(p, n, k)
        falling_product_interval_value(p, n, k) <= double_addition_carry_count(p, n)
        prime_factor_count_interval_bound(p, falling_product_interval_lower(n, k),
            n, falling_product_interval_value(p, n, k),
            double_addition_carry_count(p, n))
        falling_product_interval_central_binom_carry_bound(
            p, n, k, falling_product_interval_value(p, n, k))
    }
}

/// An interval central binomial carry-count bound gives the corresponding
/// falling-product carry-count bound.
theorem falling_product_central_binom_carry_bound_of_interval_carry_bound(
    p: Nat, n: Nat, k: Nat, value: Nat
) {
    k < n and falling_product_interval_central_binom_carry_bound(p, n, k, value)
        implies falling_product_central_binom_carry_bound(p, n, k)
} by {
    if k < n and falling_product_interval_central_binom_carry_bound(p, n, k, value) {
        prime_factor_count_interval_central_binom_carry_bound(p, n,
            falling_product_interval_lower(n, k), value)
        prime_factor_count_interval_bound(p, falling_product_interval_lower(n, k),
            n, value, double_addition_carry_count(p, n))
        prime_factor_count_interval_witness(p, falling_product_interval_lower(n, k),
            n, value)
        falling_product_interval_witness(p, n, k, value)
        falling_product_interval_witness_value_eq_prime_count_sum(p, n, k, value)
        value = falling_product_prime_count_sum(p, n, k)
        value <= double_addition_carry_count(p, n)
        falling_product_prime_count_sum(p, n, k) <= double_addition_carry_count(p, n)
        falling_product_central_binom_carry_bound(p, n, k)
    }
}

/// The canonical interval carry-count bound is equivalent to the existing
/// falling-product carry-count bound.
theorem falling_product_interval_central_binom_carry_bound_iff_carry_bound(
    p: Nat, n: Nat, k: Nat
) {
    k < n implies (
        falling_product_interval_central_binom_carry_bound(
            p, n, k, falling_product_interval_value(p, n, k)) =
        falling_product_central_binom_carry_bound(p, n, k)
    )
} by {
    if k < n {
        if falling_product_interval_central_binom_carry_bound(
            p, n, k, falling_product_interval_value(p, n, k)) {
            falling_product_central_binom_carry_bound_of_interval_carry_bound(
                p, n, k, falling_product_interval_value(p, n, k))
        }
        if falling_product_central_binom_carry_bound(p, n, k) {
            falling_product_interval_central_binom_carry_bound_of_carry_bound(p, n, k)
        }
    }
}

/// An interval central binomial valuation bound gives the corresponding
/// interval carry-count bound.
theorem falling_product_interval_central_binom_carry_bound_of_prime_bound(
    p: Nat, n: Nat, k: Nat, value: Nat
) {
    p.is_prime and k < n and
    falling_product_interval_central_binom_prime_bound(p, n, k, value)
        implies falling_product_interval_central_binom_carry_bound(p, n, k, value)
} by {
    if p.is_prime and k < n and
        falling_product_interval_central_binom_prime_bound(p, n, k, value) {
        falling_product_central_binom_prime_bound_of_interval_prime_bound(p, n, k, value)
        falling_product_central_binom_prime_bound(p, n, k)
        falling_product_central_binom_carry_bound_of_prime_bound(p, n, k)
        falling_product_central_binom_carry_bound(p, n, k)
        prime_factor_count_interval_central_binom_prime_bound(p, n,
            falling_product_interval_lower(n, k), value)
        prime_factor_count_interval_target_bound(p, falling_product_interval_lower(n, k),
            n, value, central_binom(n))
        prime_factor_count_interval_witness_of_target_bound(p,
            falling_product_interval_lower(n, k), n, value, central_binom(n))
        prime_factor_count_interval_witness(p, falling_product_interval_lower(n, k),
            n, value)
        falling_product_interval_witness(p, n, k, value)
        falling_product_interval_witness_value_eq(p, n, k, value)
        value = falling_product_interval_value(p, n, k)
        falling_product_interval_central_binom_carry_bound_of_carry_bound(p, n, k)
        falling_product_interval_central_binom_carry_bound(
            p, n, k, falling_product_interval_value(p, n, k))
        falling_product_interval_central_binom_carry_bound(p, n, k, value)
    }
}

/// An interval central binomial carry-count bound gives the corresponding
/// interval valuation bound.
theorem falling_product_interval_central_binom_prime_bound_of_carry_bound(
    p: Nat, n: Nat, k: Nat, value: Nat
) {
    p.is_prime and k < n and
    falling_product_interval_central_binom_carry_bound(p, n, k, value)
        implies falling_product_interval_central_binom_prime_bound(p, n, k, value)
} by {
    if p.is_prime and k < n and
        falling_product_interval_central_binom_carry_bound(p, n, k, value) {
        falling_product_central_binom_carry_bound_of_interval_carry_bound(p, n, k, value)
        falling_product_central_binom_carry_bound(p, n, k)
        falling_product_central_binom_prime_bound_of_carry_bound(p, n, k)
        falling_product_central_binom_prime_bound(p, n, k)
        prime_factor_count_interval_central_binom_carry_bound(p, n,
            falling_product_interval_lower(n, k), value)
        prime_factor_count_interval_bound(p, falling_product_interval_lower(n, k),
            n, value, double_addition_carry_count(p, n))
        prime_factor_count_interval_witness(p, falling_product_interval_lower(n, k),
            n, value)
        falling_product_interval_witness(p, n, k, value)
        falling_product_interval_witness_value_eq(p, n, k, value)
        value = falling_product_interval_value(p, n, k)
        falling_product_interval_central_binom_prime_bound_of_prime_bound(p, n, k)
        falling_product_interval_central_binom_prime_bound(
            p, n, k, falling_product_interval_value(p, n, k))
        falling_product_interval_central_binom_prime_bound(p, n, k, value)
    }
}

/// A binary digit-sum bound gives the corresponding interval binary bound.
theorem falling_product_interval_central_binom_binary_digit_bound_of_binary_digit_bound(
    n: Nat, k: Nat
) {
    k < n and falling_product_central_binom_binary_digit_bound(n, k)
        implies falling_product_interval_central_binom_binary_digit_bound(
            n, k, falling_product_interval_value(Nat.2, n, k))
} by {
    if k < n and falling_product_central_binom_binary_digit_bound(n, k) {
        falling_product_interval_value_witness(Nat.2, n, k)
        falling_product_interval_witness(
            Nat.2, n, k, falling_product_interval_value(Nat.2, n, k))
        falling_product_prime_count_sum(Nat.2, n, k) <= digit_sum(Nat.2, n)
        falling_product_interval_value_eq_prime_count_sum(Nat.2, n, k)
        falling_product_interval_value(Nat.2, n, k) =
            falling_product_prime_count_sum(Nat.2, n, k)
        falling_product_interval_value(Nat.2, n, k) <= digit_sum(Nat.2, n)
        prime_factor_count_interval_bound(Nat.2, falling_product_interval_lower(n, k),
            n, falling_product_interval_value(Nat.2, n, k), digit_sum(Nat.2, n))
        falling_product_interval_central_binom_binary_digit_bound(
            n, k, falling_product_interval_value(Nat.2, n, k))
    }
}

/// An interval binary digit-sum bound gives the corresponding falling-product
/// binary bound.
theorem falling_product_central_binom_binary_digit_bound_of_interval_binary_digit_bound(
    n: Nat, k: Nat, value: Nat
) {
    k < n and falling_product_interval_central_binom_binary_digit_bound(n, k, value)
        implies falling_product_central_binom_binary_digit_bound(n, k)
} by {
    if k < n and falling_product_interval_central_binom_binary_digit_bound(n, k, value) {
        prime_factor_count_interval_central_binom_binary_digit_bound(n,
            falling_product_interval_lower(n, k), value)
        prime_factor_count_interval_bound(Nat.2, falling_product_interval_lower(n, k),
            n, value, digit_sum(Nat.2, n))
        prime_factor_count_interval_witness(Nat.2, falling_product_interval_lower(n, k),
            n, value)
        falling_product_interval_witness(Nat.2, n, k, value)
        falling_product_interval_witness_value_eq_prime_count_sum(Nat.2, n, k, value)
        value = falling_product_prime_count_sum(Nat.2, n, k)
        value <= digit_sum(Nat.2, n)
        falling_product_prime_count_sum(Nat.2, n, k) <= digit_sum(Nat.2, n)
        falling_product_central_binom_binary_digit_bound(n, k)
    }
}

/// The canonical interval binary digit-sum bound is equivalent to the existing
/// falling-product binary digit-sum bound.
theorem falling_product_interval_central_binom_binary_digit_bound_iff_binary_digit_bound(
    n: Nat, k: Nat
) {
    k < n implies (
        falling_product_interval_central_binom_binary_digit_bound(
            n, k, falling_product_interval_value(Nat.2, n, k)) =
        falling_product_central_binom_binary_digit_bound(n, k)
    )
} by {
    if k < n {
        if falling_product_interval_central_binom_binary_digit_bound(
            n, k, falling_product_interval_value(Nat.2, n, k)) {
            falling_product_central_binom_binary_digit_bound_of_interval_binary_digit_bound(
                n, k, falling_product_interval_value(Nat.2, n, k))
        }
        if falling_product_central_binom_binary_digit_bound(n, k) {
            falling_product_interval_central_binom_binary_digit_bound_of_binary_digit_bound(
                n, k)
        }
    }
}

/// An interval binary digit-sum bound gives the interval carry-count bound at
/// two.
theorem falling_product_interval_central_binom_carry_bound_two_of_binary_digit_bound(
    n: Nat, k: Nat, value: Nat
) {
    k < n and falling_product_interval_central_binom_binary_digit_bound(n, k, value)
        implies falling_product_interval_central_binom_carry_bound(Nat.2, n, k, value)
} by {
    if k < n and falling_product_interval_central_binom_binary_digit_bound(n, k, value) {
        prime_factor_count_interval_central_binom_binary_digit_bound(n,
            falling_product_interval_lower(n, k), value)
        prime_factor_count_interval_bound(Nat.2, falling_product_interval_lower(n, k),
            n, value, digit_sum(Nat.2, n))
        prime_factor_count_interval_witness(Nat.2, falling_product_interval_lower(n, k),
            n, value)
        falling_product_interval_witness(Nat.2, n, k, value)
        falling_product_interval_witness_value_eq(Nat.2, n, k, value)
        value = falling_product_interval_value(Nat.2, n, k)
        falling_product_central_binom_binary_digit_bound_of_interval_binary_digit_bound(
            n, k, value)
        falling_product_central_binom_binary_digit_bound(n, k)
        falling_product_central_binom_carry_bound_two_of_binary_digit_bound(n, k)
        falling_product_central_binom_carry_bound(Nat.2, n, k)
        falling_product_interval_central_binom_carry_bound_of_carry_bound(Nat.2, n, k)
        falling_product_interval_central_binom_carry_bound(
            Nat.2, n, k, falling_product_interval_value(Nat.2, n, k))
        falling_product_interval_central_binom_carry_bound(Nat.2, n, k, value)
    }
}

/// An interval carry-count bound at two gives the interval binary digit-sum
/// bound.
theorem falling_product_interval_central_binom_binary_digit_bound_of_carry_bound_two(
    n: Nat, k: Nat, value: Nat
) {
    k < n and falling_product_interval_central_binom_carry_bound(Nat.2, n, k, value)
        implies falling_product_interval_central_binom_binary_digit_bound(n, k, value)
} by {
    if k < n and falling_product_interval_central_binom_carry_bound(Nat.2, n, k, value) {
        prime_factor_count_interval_central_binom_carry_bound(Nat.2, n,
            falling_product_interval_lower(n, k), value)
        prime_factor_count_interval_bound(Nat.2, falling_product_interval_lower(n, k),
            n, value, double_addition_carry_count(Nat.2, n))
        prime_factor_count_interval_witness(Nat.2, falling_product_interval_lower(n, k),
            n, value)
        falling_product_interval_witness(Nat.2, n, k, value)
        falling_product_interval_witness_value_eq(Nat.2, n, k, value)
        value = falling_product_interval_value(Nat.2, n, k)
        falling_product_central_binom_carry_bound_of_interval_carry_bound(
            Nat.2, n, k, value)
        falling_product_central_binom_carry_bound(Nat.2, n, k)
        falling_product_central_binom_binary_digit_bound_of_carry_bound_two(n, k)
        falling_product_central_binom_binary_digit_bound(n, k)
        falling_product_interval_central_binom_binary_digit_bound_of_binary_digit_bound(
            n, k)
        falling_product_interval_central_binom_binary_digit_bound(
            n, k, falling_product_interval_value(Nat.2, n, k))
        falling_product_interval_central_binom_binary_digit_bound(n, k, value)
    }
}

/// The interval binary digit-sum bound is equivalent to the interval
/// carry-count bound at two.
theorem falling_product_interval_central_binom_binary_digit_bound_iff_carry_bound_two(
    n: Nat, k: Nat
) {
    k < n implies (
        falling_product_interval_central_binom_binary_digit_bound(
            n, k, falling_product_interval_value(Nat.2, n, k)) =
        falling_product_interval_central_binom_carry_bound(
            Nat.2, n, k, falling_product_interval_value(Nat.2, n, k))
    )
} by {
    if k < n {
        if falling_product_interval_central_binom_binary_digit_bound(
            n, k, falling_product_interval_value(Nat.2, n, k)) {
            falling_product_interval_central_binom_carry_bound_two_of_binary_digit_bound(
                n, k, falling_product_interval_value(Nat.2, n, k))
        }
        if falling_product_interval_central_binom_carry_bound(
            Nat.2, n, k, falling_product_interval_value(Nat.2, n, k)) {
            falling_product_interval_central_binom_binary_digit_bound_of_carry_bound_two(
                n, k, falling_product_interval_value(Nat.2, n, k))
        }
    }
}

/// A primewise central binomial valuation bound gives the interval primewise
/// central binomial valuation bound.
theorem falling_product_interval_central_binom_primewise_bound_of_primewise_bound(
    n: Nat, k: Nat
) {
    k < n and falling_product_central_binom_primewise_bound(n, k)
        implies falling_product_interval_central_binom_primewise_bound(n, k)
} by {
    if k < n and falling_product_central_binom_primewise_bound(n, k) {
        forall(p: Nat) {
            if p.is_prime {
                let h: Bool = p.is_prime implies falling_product_central_binom_prime_bound(
                    p, n, k)
                h
                falling_product_central_binom_prime_bound(p, n, k)
                falling_product_interval_central_binom_prime_bound_of_prime_bound(
                    p, n, k)
                falling_product_interval_central_binom_prime_bound(
                    p, n, k, falling_product_interval_value(p, n, k))
            }
        }
        falling_product_interval_central_binom_primewise_bound(n, k)
    }
}

/// An interval primewise central binomial valuation bound gives the existing
/// primewise central binomial valuation bound.
theorem falling_product_central_binom_primewise_bound_of_interval_primewise_bound(
    n: Nat, k: Nat
) {
    k < n and falling_product_interval_central_binom_primewise_bound(n, k)
        implies falling_product_central_binom_primewise_bound(n, k)
} by {
    if k < n and falling_product_interval_central_binom_primewise_bound(n, k) {
        forall(p: Nat) {
            if p.is_prime {
                let h: Bool = p.is_prime implies falling_product_interval_central_binom_prime_bound(
                    p, n, k, falling_product_interval_value(p, n, k))
                h
                falling_product_interval_central_binom_prime_bound(
                    p, n, k, falling_product_interval_value(p, n, k))
                falling_product_central_binom_prime_bound_of_interval_prime_bound(
                    p, n, k, falling_product_interval_value(p, n, k))
                falling_product_central_binom_prime_bound(p, n, k)
            }
        }
        falling_product_central_binom_primewise_bound(n, k)
    }
}

/// The interval primewise central binomial valuation bound is equivalent to
/// the existing primewise valuation bound.
theorem falling_product_interval_central_binom_primewise_bound_iff_primewise_bound(
    n: Nat, k: Nat
) {
    k < n implies (
        falling_product_interval_central_binom_primewise_bound(n, k) =
        falling_product_central_binom_primewise_bound(n, k)
    )
} by {
    if k < n {
        if falling_product_interval_central_binom_primewise_bound(n, k) {
            falling_product_central_binom_primewise_bound_of_interval_primewise_bound(n, k)
        }
        if falling_product_central_binom_primewise_bound(n, k) {
            falling_product_interval_central_binom_primewise_bound_of_primewise_bound(n, k)
        }
    }
}

/// An interval primewise central binomial valuation bound gives the interval
/// valuation bound at a particular prime.
theorem falling_product_interval_central_binom_prime_bound_of_interval_primewise_bound(
    p: Nat, n: Nat, k: Nat
) {
    p.is_prime and falling_product_interval_central_binom_primewise_bound(n, k)
        implies falling_product_interval_central_binom_prime_bound(
            p, n, k, falling_product_interval_value(p, n, k))
} by {
    if p.is_prime and falling_product_interval_central_binom_primewise_bound(n, k) {
        let h: Bool = p.is_prime implies falling_product_interval_central_binom_prime_bound(
            p, n, k, falling_product_interval_value(p, n, k))
        h
        falling_product_interval_central_binom_prime_bound(
            p, n, k, falling_product_interval_value(p, n, k))
    }
}

/// An interval central binomial valuation bound gives the corresponding
/// prime-count inequality for the falling product.
theorem count_prime_factor_falling_product_le_central_binom_of_interval_prime_bound(
    p: Nat, n: Nat, k: Nat, value: Nat
) {
    k < n and falling_product_interval_central_binom_prime_bound(p, n, k, value)
        implies count_prime_factor(p, falling_product(n, k)) <= count_prime_factor(
            p, central_binom(n))
} by {
    if k < n and falling_product_interval_central_binom_prime_bound(p, n, k, value) {
        falling_product_central_binom_prime_bound_of_interval_prime_bound(p, n, k, value)
        falling_product_central_binom_prime_bound(p, n, k)
        count_prime_factor_falling_product_le_central_binom_of_prime_bound(p, n, k)
        count_prime_factor(p, falling_product(n, k)) <= count_prime_factor(
            p, central_binom(n))
    }
}

/// Interval primewise central binomial valuation bounds give the prime-count
/// inequalities needed for central-binomial divisibility.
theorem count_prime_factor_falling_product_le_central_binom_of_interval_primewise_bound(
    p: Nat, n: Nat, k: Nat
) {
    p.is_prime and k < n and
    falling_product_interval_central_binom_primewise_bound(n, k)
        implies count_prime_factor(p, falling_product(n, k)) <= count_prime_factor(
            p, central_binom(n))
} by {
    if p.is_prime and k < n and
        falling_product_interval_central_binom_primewise_bound(n, k) {
        falling_product_central_binom_primewise_bound_of_interval_primewise_bound(n, k)
        falling_product_central_binom_primewise_bound(n, k)
        count_prime_factor_falling_product_le_central_binom_of_primewise_bound(
            p, n, k)
        count_prime_factor(p, falling_product(n, k)) <= count_prime_factor(
            p, central_binom(n))
    }
}

/// A primewise central binomial carry-count bound gives the interval primewise
/// carry-count bound.
theorem falling_product_interval_central_binom_carrywise_bound_of_carrywise_bound(
    n: Nat, k: Nat
) {
    k < n and falling_product_central_binom_carrywise_bound(n, k)
        implies falling_product_interval_central_binom_carrywise_bound(n, k)
} by {
    if k < n and falling_product_central_binom_carrywise_bound(n, k) {
        forall(p: Nat) {
            if p.is_prime {
                let h: Bool = p.is_prime implies falling_product_central_binom_carry_bound(
                    p, n, k)
                h
                falling_product_central_binom_carry_bound(p, n, k)
                falling_product_interval_central_binom_carry_bound_of_carry_bound(
                    p, n, k)
                falling_product_interval_central_binom_carry_bound(
                    p, n, k, falling_product_interval_value(p, n, k))
            }
        }
        falling_product_interval_central_binom_carrywise_bound(n, k)
    }
}

/// An interval primewise carry-count bound gives the existing primewise
/// central binomial carry-count bound.
theorem falling_product_central_binom_carrywise_bound_of_interval_carrywise_bound(
    n: Nat, k: Nat
) {
    k < n and falling_product_interval_central_binom_carrywise_bound(n, k)
        implies falling_product_central_binom_carrywise_bound(n, k)
} by {
    if k < n and falling_product_interval_central_binom_carrywise_bound(n, k) {
        forall(p: Nat) {
            if p.is_prime {
                let h: Bool = p.is_prime implies falling_product_interval_central_binom_carry_bound(
                    p, n, k, falling_product_interval_value(p, n, k))
                h
                falling_product_interval_central_binom_carry_bound(
                    p, n, k, falling_product_interval_value(p, n, k))
                falling_product_central_binom_carry_bound_of_interval_carry_bound(
                    p, n, k, falling_product_interval_value(p, n, k))
                falling_product_central_binom_carry_bound(p, n, k)
            }
        }
        falling_product_central_binom_carrywise_bound(n, k)
    }
}

/// The interval primewise carry-count bound is equivalent to the existing
/// primewise carry-count bound.
theorem falling_product_interval_central_binom_carrywise_bound_iff_carrywise_bound(
    n: Nat, k: Nat
) {
    k < n implies (
        falling_product_interval_central_binom_carrywise_bound(n, k) =
        falling_product_central_binom_carrywise_bound(n, k)
    )
} by {
    if k < n {
        if falling_product_interval_central_binom_carrywise_bound(n, k) {
            falling_product_central_binom_carrywise_bound_of_interval_carrywise_bound(n, k)
        }
        if falling_product_central_binom_carrywise_bound(n, k) {
            falling_product_interval_central_binom_carrywise_bound_of_carrywise_bound(n, k)
        }
    }
}

/// An interval primewise carry-count bound gives the interval carry-count
/// bound at a particular prime.
theorem falling_product_interval_central_binom_carry_bound_of_interval_carrywise_bound(
    p: Nat, n: Nat, k: Nat
) {
    p.is_prime and falling_product_interval_central_binom_carrywise_bound(n, k)
        implies falling_product_interval_central_binom_carry_bound(
            p, n, k, falling_product_interval_value(p, n, k))
} by {
    if p.is_prime and falling_product_interval_central_binom_carrywise_bound(n, k) {
        let h: Bool = p.is_prime implies falling_product_interval_central_binom_carry_bound(
            p, n, k, falling_product_interval_value(p, n, k))
        h
        falling_product_interval_central_binom_carry_bound(
            p, n, k, falling_product_interval_value(p, n, k))
    }
}

/// Interval primewise central binomial valuation bounds imply interval
/// primewise carry-count bounds.
theorem falling_product_interval_central_binom_carrywise_bound_of_primewise_bound(
    n: Nat, k: Nat
) {
    k < n and falling_product_interval_central_binom_primewise_bound(n, k)
        implies falling_product_interval_central_binom_carrywise_bound(n, k)
} by {
    if k < n and falling_product_interval_central_binom_primewise_bound(n, k) {
        forall(p: Nat) {
            if p.is_prime {
                let h: Bool = p.is_prime implies falling_product_interval_central_binom_prime_bound(
                    p, n, k, falling_product_interval_value(p, n, k))
                h
                falling_product_interval_central_binom_prime_bound(
                    p, n, k, falling_product_interval_value(p, n, k))
                falling_product_interval_central_binom_carry_bound_of_prime_bound(
                    p, n, k, falling_product_interval_value(p, n, k))
                falling_product_interval_central_binom_carry_bound(
                    p, n, k, falling_product_interval_value(p, n, k))
            }
        }
        falling_product_interval_central_binom_carrywise_bound(n, k)
    }
}

/// Interval primewise carry-count bounds imply interval primewise central
/// binomial valuation bounds.
theorem falling_product_interval_central_binom_primewise_bound_of_carrywise_bound(
    n: Nat, k: Nat
) {
    k < n and falling_product_interval_central_binom_carrywise_bound(n, k)
        implies falling_product_interval_central_binom_primewise_bound(n, k)
} by {
    if k < n and falling_product_interval_central_binom_carrywise_bound(n, k) {
        forall(p: Nat) {
            if p.is_prime {
                let h: Bool = p.is_prime implies falling_product_interval_central_binom_carry_bound(
                    p, n, k, falling_product_interval_value(p, n, k))
                h
                falling_product_interval_central_binom_carry_bound(
                    p, n, k, falling_product_interval_value(p, n, k))
                falling_product_interval_central_binom_prime_bound_of_carry_bound(
                    p, n, k, falling_product_interval_value(p, n, k))
                falling_product_interval_central_binom_prime_bound(
                    p, n, k, falling_product_interval_value(p, n, k))
            }
        }
        falling_product_interval_central_binom_primewise_bound(n, k)
    }
}

/// The interval primewise valuation bound and interval primewise carry-count
/// bound are equivalent.
theorem falling_product_interval_central_binom_primewise_bound_iff_carrywise_bound(
    n: Nat, k: Nat
) {
    k < n implies (
        falling_product_interval_central_binom_primewise_bound(n, k) =
        falling_product_interval_central_binom_carrywise_bound(n, k)
    )
} by {
    if k < n {
        if falling_product_interval_central_binom_primewise_bound(n, k) {
            falling_product_interval_central_binom_carrywise_bound_of_primewise_bound(n, k)
        }
        if falling_product_interval_central_binom_carrywise_bound(n, k) {
            falling_product_interval_central_binom_primewise_bound_of_carrywise_bound(n, k)
        }
    }
}

/// Interval primewise central binomial valuation bounds imply divisibility of
/// the central binomial coefficient by the falling product.
theorem falling_product_divides_central_binom_of_interval_primewise_bound(
    n: Nat, k: Nat
) {
    k < n and falling_product_interval_central_binom_primewise_bound(n, k)
        implies falling_product(n, k).divides(central_binom(n))
} by {
    if k < n and falling_product_interval_central_binom_primewise_bound(n, k) {
        falling_product_central_binom_primewise_bound_of_interval_primewise_bound(n, k)
        falling_product_central_binom_primewise_bound(n, k)
        falling_product_divides_central_binom_of_primewise_bound(n, k)
        falling_product(n, k).divides(central_binom(n))
    }
}

/// Interval primewise central binomial carry-count bounds imply divisibility
/// of the central binomial coefficient by the falling product.
theorem falling_product_divides_central_binom_of_interval_carrywise_bound(
    n: Nat, k: Nat
) {
    k < n and falling_product_interval_central_binom_carrywise_bound(n, k)
        implies falling_product(n, k).divides(central_binom(n))
} by {
    if k < n and falling_product_interval_central_binom_carrywise_bound(n, k) {
        falling_product_central_binom_carrywise_bound_of_interval_carrywise_bound(n, k)
        falling_product_central_binom_carrywise_bound(n, k)
        falling_product_divides_central_binom_of_carrywise_bound(n, k)
        falling_product(n, k).divides(central_binom(n))
    }
}

/// Falling-product divisibility of the central binomial coefficient gives the
/// interval primewise central binomial valuation bounds.
theorem falling_product_interval_central_binom_primewise_bound_of_divides(
    n: Nat, k: Nat
) {
    k < n and falling_product(n, k).divides(central_binom(n))
        implies falling_product_interval_central_binom_primewise_bound(n, k)
} by {
    if k < n and falling_product(n, k).divides(central_binom(n)) {
        falling_product_central_binom_primewise_bound_of_divides(n, k)
        falling_product_central_binom_primewise_bound(n, k)
        falling_product_interval_central_binom_primewise_bound_of_primewise_bound(n, k)
        falling_product_interval_central_binom_primewise_bound(n, k)
    }
}

/// Falling-product divisibility of the central binomial coefficient gives the
/// interval primewise carry-count bounds.
theorem falling_product_interval_central_binom_carrywise_bound_of_divides(
    n: Nat, k: Nat
) {
    k < n and falling_product(n, k).divides(central_binom(n))
        implies falling_product_interval_central_binom_carrywise_bound(n, k)
} by {
    if k < n and falling_product(n, k).divides(central_binom(n)) {
        falling_product_central_binom_carrywise_bound_of_divides(n, k)
        falling_product_central_binom_carrywise_bound(n, k)
        falling_product_interval_central_binom_carrywise_bound_of_carrywise_bound(n, k)
        falling_product_interval_central_binom_carrywise_bound(n, k)
    }
}

/// A positive falling product divides the central binomial coefficient iff the
/// interval primewise central binomial valuation bounds hold.
theorem falling_product_divides_central_binom_iff_interval_primewise_bound(
    n: Nat, k: Nat
) {
    k < n implies (
        falling_product(n, k).divides(central_binom(n)) =
        falling_product_interval_central_binom_primewise_bound(n, k)
    )
} by {
    if k < n {
        let divides_central: Bool = falling_product(n, k).divides(central_binom(n))
        let primewise_bound: Bool = falling_product_central_binom_primewise_bound(n, k)
        let interval_bound: Bool =
            falling_product_interval_central_binom_primewise_bound(n, k)
        falling_product_divides_central_binom_iff_primewise_bound(n, k)
        divides_central = primewise_bound
        falling_product_interval_central_binom_primewise_bound_iff_primewise_bound(n, k)
        interval_bound = primewise_bound
        primewise_bound = interval_bound
        divides_central = interval_bound
        falling_product(n, k).divides(central_binom(n)) =
            falling_product_interval_central_binom_primewise_bound(n, k)
    }
}

/// A positive falling product divides the central binomial coefficient iff the
/// interval primewise carry-count bounds hold.
theorem falling_product_divides_central_binom_iff_interval_carrywise_bound(
    n: Nat, k: Nat
) {
    k < n implies (
        falling_product(n, k).divides(central_binom(n)) =
        falling_product_interval_central_binom_carrywise_bound(n, k)
    )
} by {
    if k < n {
        let divides_central: Bool = falling_product(n, k).divides(central_binom(n))
        let carrywise_bound: Bool = falling_product_central_binom_carrywise_bound(n, k)
        let interval_bound: Bool =
            falling_product_interval_central_binom_carrywise_bound(n, k)
        falling_product_divides_central_binom_iff_carrywise_bound(n, k)
        divides_central = carrywise_bound
        falling_product_interval_central_binom_carrywise_bound_iff_carrywise_bound(n, k)
        interval_bound = carrywise_bound
        carrywise_bound = interval_bound
        divides_central = interval_bound
        falling_product(n, k).divides(central_binom(n)) =
            falling_product_interval_central_binom_carrywise_bound(n, k)
    }
}
