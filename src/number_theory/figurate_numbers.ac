// The figurate numbers: triangular, square, pentagonal, and octagonal, and
// the relations between them.
//
// The polygonal numbers are the classical "figurate" numbers: the n-th
// triangular number T_n = n(n+1)/2, the n-th square number S_n = n^2, the
// n-th pentagonal number P_n = n(3n-1)/2, and the n-th octagonal number
// O_n = n(3n-2).  Each counts the dots in an arrangement of nested polygons
// of side n.
//
// The triangular numbers are the sums 0 + 1 + ... + n, which the library
// already carries as `triangular` in the combinatorics package with the
// closed form 2 * T_n = n(n+1) (`triangular_doubled`); the closed form
// T_n = n(n+1)/2 is identified with that sum by `figurate_triangular_closed`.
// The square-triangular connection below states that every square is the sum
// of two consecutive triangular numbers, and the summation identity states
// that the sum of the first n triangular numbers is n(n+1)(n+2)/6.
//
// Division by two appears only in the literal closed-form definitions
// `figurate_triangular` and `figurate_pentagonal`; the proofs work with the
// doubled forms, e.g. 2 * T_n = n(n+1) and 2 * P_n = n(3n-1).

from nat import Nat, add_comm, add_assoc, add_zero_right, add_zero_left, add_suc_right,
    add_one_right, mul_comm, mul_assoc, distrib_left, distrib_right, mul_one_right,
    mul_one_left, mul_zero_right, mul_zero_left, mul_two_left, mul_cancel_left,
    suc_sub_one, one_plus_one, div_mul
from combinatorics import triangular, triangular_zero, triangular_suc, triangular_doubled
from data.nat.nat_range_sum import range_sum, range_sum_suc, range_sum_zero

numerals Nat

// ---------------------------------------------------------------------------
// The definitions.
// ---------------------------------------------------------------------------

/// The n-th triangular number as the closed form `T_n = n(n+1)/2`.
///
/// The literal closed form, using natural division.  The library's
/// `triangular` (the sum `0 + 1 + ... + n`) is identified with it by
/// `figurate_triangular_closed`, and `triangular_doubled` gives the
/// division-free form `2 * T_n = n(n+1)`.
define figurate_triangular(n: Nat) -> Nat {
    (n * (n + Nat.1)).div(Nat.2)
}

/// The n-th square number: `S_n = n^2`.
define figurate_square(n: Nat) -> Nat {
    n * n
}

/// The n-th pentagonal number as the closed form `P_n = n(3n-1)/2`.
define figurate_pentagonal(n: Nat) -> Nat {
    (n * (Nat.3 * n - Nat.1)).div(Nat.2)
}

/// Twice the n-th pentagonal number: `2 * P_n = n * (3n - 1)`.
///
/// The doubled form of the pentagonal closed form, avoiding division.
define figurate_pentagonal_doubled(n: Nat) -> Nat {
    n * (Nat.3 * n - Nat.1)
}

/// The n-th octagonal number: `O_n = n(3n - 2)`.
define figurate_octagonal(n: Nat) -> Nat {
    n * (Nat.3 * n - Nat.2)
}

// ---------------------------------------------------------------------------
// The triangular numbers.
// ---------------------------------------------------------------------------

/// The closed form `T_n = n(n+1)/2` is the sum `0 + 1 + ... + n`:
/// `figurate_triangular(n) = triangular(n)`.
theorem figurate_triangular_closed(n: Nat) {
    figurate_triangular(n) = triangular(n)
} by {
    figurate_triangular(n) = (n * (n + Nat.1)).div(Nat.2)
    triangular_doubled(n)
    Nat.2 * triangular(n) = n * (n + Nat.1)
    div_mul(triangular(n), Nat.2)
    (triangular(n) * Nat.2).div(Nat.2) = triangular(n)
    (n * (n + Nat.1)).div(Nat.2) = triangular(n)
}

/// `T_1 = 1`.
theorem figurate_triangular_one {
    figurate_triangular(Nat.1) = Nat.1
} by {
    figurate_triangular(Nat.1) = (Nat.1 * (Nat.1 + Nat.1)).div(Nat.2)
    Nat.1 + Nat.1 = Nat.2
    (Nat.1 * Nat.2).div(Nat.2) = Nat.1
}

/// `T_2 = 3`.
theorem figurate_triangular_two {
    figurate_triangular(Nat.2) = Nat.3
} by {
    figurate_triangular(Nat.2) = (Nat.2 * (Nat.2 + Nat.1)).div(Nat.2)
    Nat.2 + Nat.1 = Nat.3
    Nat.2 * Nat.3 = Nat.6
    (Nat.3 * Nat.2).div(Nat.2) = Nat.3
    (Nat.2 * Nat.3).div(Nat.2) = Nat.3
}

/// `T_3 = 6`.
theorem figurate_triangular_three {
    figurate_triangular(Nat.3) = Nat.6
} by {
    figurate_triangular(Nat.3) = (Nat.3 * (Nat.3 + Nat.1)).div(Nat.2)
    Nat.3 + Nat.1 = Nat.4
    Nat.3 * Nat.4 = Nat.12
    (Nat.6 * Nat.2).div(Nat.2) = Nat.6
    (Nat.3 * Nat.4).div(Nat.2) = Nat.6
}

/// `T_4 = 10`.
theorem figurate_triangular_four {
    figurate_triangular(Nat.4) = Nat.10
} by {
    figurate_triangular(Nat.4) = (Nat.4 * (Nat.4 + Nat.1)).div(Nat.2)
    Nat.4 + Nat.1 = Nat.5
    Nat.4 * Nat.5 = Nat.20
    (Nat.10 * Nat.2).div(Nat.2) = Nat.10
    (Nat.4 * Nat.5).div(Nat.2) = Nat.10
}

// ---------------------------------------------------------------------------
// The square numbers.
// ---------------------------------------------------------------------------

/// `S_1 = 1`.
theorem figurate_square_one {
    figurate_square(Nat.1) = Nat.1
}

/// `S_2 = 4`.
theorem figurate_square_two {
    figurate_square(Nat.2) = Nat.4
}

/// `S_3 = 9`.
theorem figurate_square_three {
    figurate_square(Nat.3) = Nat.9
}

/// `S_4 = 16`.
theorem figurate_square_four {
    figurate_square(Nat.4) = Nat.16
}

// ---------------------------------------------------------------------------
// The square-triangular connection: every square is the sum of two
// consecutive triangular numbers, T_n + T_{n-1} = n^2.
// ---------------------------------------------------------------------------

/// Two consecutive triangular numbers sum to a square:
/// `T_n + T_{n-1} = n^2`.
///
/// The recurrence `T_{k+1} = k+1 + T_k` and the closed form
/// `2 * T_k = k(k+1)` give `T_{k+1} + T_k = (k+1) + 2T_k = (k+1)(k+1)`
/// directly, so the identity is proved by induction with the step needing no
/// induction hypothesis (the case k = 0 is the base case, where the
/// subtraction `0 - 1` is truncated to zero and `T_0 = 0`).
theorem figurate_triangular_pair_sum(n: Nat) {
    triangular(n) + triangular(n - Nat.1) = n * n
} by {
    define p(k: Nat) -> Bool {
        triangular(k) + triangular(k - Nat.1) = k * k
    }
    triangular_zero
    triangular(Nat.0) = Nat.0
    Nat.0 - Nat.1 = Nat.0
    triangular(Nat.0 - Nat.1) = triangular(Nat.0)
    triangular(Nat.0) + triangular(Nat.0 - Nat.1) = Nat.0 * Nat.0
    p(Nat.0)
    forall(k: Nat) {
        if p(k) {
            triangular_suc(k)
            triangular(k.suc) = k.suc + triangular(k)
            suc_sub_one(k)
            k.suc - Nat.1 = k
            triangular(k.suc - Nat.1) = triangular(k)
            triangular_doubled(k)
            Nat.2 * triangular(k) = k * (k + Nat.1)
            mul_two_left(triangular(k))
            Nat.2 * triangular(k) = triangular(k) + triangular(k)
            (k.suc + triangular(k)) + triangular(k) = k.suc + (triangular(k) + triangular(k))
            k.suc + (triangular(k) + triangular(k)) = k.suc + (Nat.2 * triangular(k))
            k.suc + (Nat.2 * triangular(k)) = k.suc + (k * (k + Nat.1))
            k + Nat.1 = k.suc
            k * k.suc = k * (k + Nat.1)
            k.suc + (k * k.suc) = k.suc * k.suc
            triangular(k.suc) + triangular(k.suc - Nat.1) = k.suc * k.suc
            p(k.suc)
        }
    }
    p(Nat.0) and forall(k: Nat) { p(k) implies p(k.suc) }
    Nat.induction(p)
    p(n)
}

/// Every square is the sum of two consecutive triangular numbers:
/// `T_n + T_{n-1} = S_n` in the closed forms of the figurate numbers.
theorem figurate_square_triangular(n: Nat) {
    figurate_triangular(n) + figurate_triangular(n - Nat.1) = figurate_square(n)
} by {
    figurate_triangular_closed(n)
    figurate_triangular(n) = triangular(n)
    figurate_triangular_closed(n - Nat.1)
    figurate_triangular(n - Nat.1) = triangular(n - Nat.1)
    figurate_square(n) = n * n
    figurate_triangular_pair_sum(n)
    triangular(n) + triangular(n - Nat.1) = n * n
    figurate_triangular(n) + figurate_triangular(n - Nat.1) = n * n
    figurate_triangular(n) + figurate_triangular(n - Nat.1) = figurate_square(n)
}

// ---------------------------------------------------------------------------
// The pentagonal numbers.
// ---------------------------------------------------------------------------

/// `P_1 = 1`.
theorem figurate_pentagonal_one {
    figurate_pentagonal(Nat.1) = Nat.1
} by {
    figurate_pentagonal(Nat.1) = (Nat.1 * (Nat.3 * Nat.1 - Nat.1)).div(Nat.2)
    Nat.3 * Nat.1 = Nat.3
    Nat.3 - Nat.1 = Nat.2
    (Nat.1 * Nat.2).div(Nat.2) = Nat.1
}

/// `P_2 = 5`.
theorem figurate_pentagonal_two {
    figurate_pentagonal(Nat.2) = Nat.5
} by {
    figurate_pentagonal(Nat.2) = (Nat.2 * (Nat.3 * Nat.2 - Nat.1)).div(Nat.2)
    Nat.3 * Nat.2 = Nat.6
    Nat.6 - Nat.1 = Nat.5
    (Nat.5 * Nat.2).div(Nat.2) = Nat.5
    (Nat.2 * Nat.5).div(Nat.2) = Nat.5
}

/// `P_3 = 12`.
theorem figurate_pentagonal_three {
    figurate_pentagonal(Nat.3) = Nat.12
} by {
    figurate_pentagonal(Nat.3) = (Nat.3 * (Nat.3 * Nat.3 - Nat.1)).div(Nat.2)
    Nat.3 * Nat.3 = Nat.9
    Nat.9 - Nat.1 = Nat.8
    Nat.3 * Nat.8 = Nat.24
    Nat.24 = Nat.12 * Nat.2
    div_mul(Nat.12, Nat.2)
    (Nat.12 * Nat.2).div(Nat.2) = Nat.12
    (Nat.3 * Nat.8).div(Nat.2) = Nat.12
}

/// `P_4 = 22`.
theorem figurate_pentagonal_four {
    figurate_pentagonal(Nat.4) = Nat.22
} by {
    figurate_pentagonal(Nat.4) = (Nat.4 * (Nat.3 * Nat.4 - Nat.1)).div(Nat.2)
    Nat.3 * Nat.4 = Nat.12
    suc_sub_one(Nat.11)
    Nat.11.suc - Nat.1 = Nat.11
    Nat.11.suc = Nat.12
    Nat.12 - Nat.1 = Nat.11
    Nat.4 * Nat.11 = Nat.44
    Nat.44 = Nat.22 * Nat.2
    div_mul(Nat.22, Nat.2)
    (Nat.22 * Nat.2).div(Nat.2) = Nat.22
    (Nat.4 * Nat.11).div(Nat.2) = Nat.22
}

// ---------------------------------------------------------------------------
// The sum of the triangular numbers.
// ---------------------------------------------------------------------------

/// The sum of the first n triangular numbers: `T_0 + T_1 + ... + T_n`.
///
/// The `T_0 = 0` term contributes nothing, so this is the classical sum
/// `sum_{k=1}^{n} T_k` of the first n positive triangular numbers.
define figurate_triangular_sum(n: Nat) -> Nat {
    range_sum(triangular, n.suc)
}

/// Six times a triangular number: `6 * T_n = 3 * n * (n + 1)`.
///
/// From the closed form `2 * T_n = n(n+1)` of `triangular_doubled`,
/// multiplying both sides by three gives `6 * T_n = 3 * n(n+1)`.
theorem figurate_triangular_six(n: Nat) {
    Nat.6 * triangular(n) = Nat.3 * (n * (n + Nat.1))
} by {
    triangular_doubled(n)
    Nat.2 * triangular(n) = n * (n + Nat.1)
    Nat.6 = Nat.3 * Nat.2
    Nat.6 * triangular(n) = (Nat.3 * Nat.2) * triangular(n)
    mul_assoc(Nat.3, Nat.2, triangular(n))
    (Nat.3 * Nat.2) * triangular(n) = Nat.3 * (Nat.2 * triangular(n))
    Nat.3 * (Nat.2 * triangular(n)) = Nat.3 * (n * (n + Nat.1))
    Nat.6 * triangular(n) = Nat.3 * (n * (n + Nat.1))
}

/// The sum of the first n triangular numbers is `n(n+1)(n+2)/6`:
/// `6 * (T_0 + ... + T_n) = n(n+1)(n+2)`.
///
/// Induction on n; the step uses `T_{k+1} = (k+1)(k+2)/2`, i.e.
/// `6 * T_{k+1} = 3(k+1)(k+2)`, and factors `(k+1)(k+2)` out of
/// `k(k+1)(k+2) + 3(k+1)(k+2)`.
theorem figurate_triangular_sum_closed(n: Nat) {
    Nat.6 * range_sum(triangular, n.suc) = n * (n + Nat.1) * (n + Nat.2)
} by {
    define p(k: Nat) -> Bool {
        Nat.6 * range_sum(triangular, k.suc) = k * (k + Nat.1) * (k + Nat.2)
    }
    range_sum_suc(triangular, Nat.0)
    range_sum(triangular, Nat.1) = range_sum(triangular, Nat.0) + triangular(Nat.0)
    range_sum_zero(triangular)
    range_sum(triangular, Nat.0) = Nat.0
    triangular_zero
    triangular(Nat.0) = Nat.0
    range_sum(triangular, Nat.1) = Nat.0
    Nat.6 * range_sum(triangular, Nat.1) = Nat.0
    Nat.0 * (Nat.0 + Nat.1) * (Nat.0 + Nat.2) = Nat.0
    Nat.6 * range_sum(triangular, Nat.1) = Nat.0 * (Nat.0 + Nat.1) * (Nat.0 + Nat.2)
    p(Nat.0)
    forall(k: Nat) {
        if p(k) {
            range_sum_suc(triangular, k.suc)
            range_sum(triangular, k.suc.suc) =
                range_sum(triangular, k.suc) + triangular(k.suc)
            Nat.6 * range_sum(triangular, k.suc.suc) =
                Nat.6 * (range_sum(triangular, k.suc) + triangular(k.suc))
            distrib_left(Nat.6, range_sum(triangular, k.suc), triangular(k.suc))
            Nat.6 * (range_sum(triangular, k.suc) + triangular(k.suc)) =
                Nat.6 * range_sum(triangular, k.suc) + Nat.6 * triangular(k.suc)
            Nat.6 * range_sum(triangular, k.suc) + Nat.6 * triangular(k.suc) =
                k * (k + Nat.1) * (k + Nat.2) + Nat.6 * triangular(k.suc)
            figurate_triangular_six(k.suc)
            Nat.6 * triangular(k.suc) = Nat.3 * (k.suc * (k.suc + Nat.1))
            k.suc * (k.suc + Nat.1) = (k + Nat.1) * (k + Nat.2)
            Nat.6 * triangular(k.suc) = Nat.3 * ((k + Nat.1) * (k + Nat.2))
            k * (k + Nat.1) * (k + Nat.2) + Nat.6 * triangular(k.suc) =
                k * (k + Nat.1) * (k + Nat.2) + Nat.3 * ((k + Nat.1) * (k + Nat.2))
            k * ((k + Nat.1) * (k + Nat.2)) + Nat.3 * ((k + Nat.1) * (k + Nat.2)) =
                (k + Nat.3) * ((k + Nat.1) * (k + Nat.2))
            (k + Nat.3) * ((k + Nat.1) * (k + Nat.2)) =
                k.suc * (k.suc + Nat.1) * (k.suc + Nat.2)
            Nat.6 * range_sum(triangular, k.suc.suc) =
                k.suc * (k.suc + Nat.1) * (k.suc + Nat.2)
            p(k.suc)
        }
    }
    p(Nat.0) and forall(k: Nat) { p(k) implies p(k.suc) }
    Nat.induction(p)
    p(n)
    Nat.6 * range_sum(triangular, n.suc) = n * (n + Nat.1) * (n + Nat.2)
}

/// The sum of the first n triangular numbers in the closed forms of the
/// figurate numbers: `6 * (T_0 + ... + T_n) = n(n+1)(n+2)`.
theorem figurate_triangular_sum_figurate(n: Nat) {
    Nat.6 * figurate_triangular_sum(n) = n * (n + Nat.1) * (n + Nat.2)
} by {
    figurate_triangular_sum(n) = range_sum(triangular, n.suc)
    figurate_triangular_sum_closed(n)
    Nat.6 * range_sum(triangular, n.suc) = n * (n + Nat.1) * (n + Nat.2)
    Nat.6 * figurate_triangular_sum(n) = n * (n + Nat.1) * (n + Nat.2)
}

// ---------------------------------------------------------------------------
// The octagonal numbers.
// ---------------------------------------------------------------------------

/// `O_1 = 1`.
theorem figurate_octagonal_one {
    figurate_octagonal(Nat.1) = Nat.1
} by {
    figurate_octagonal(Nat.1) = Nat.1 * (Nat.3 * Nat.1 - Nat.2)
    Nat.3 * Nat.1 = Nat.3
    Nat.3 - Nat.2 = Nat.1
    Nat.1 * Nat.1 = Nat.1
}

/// `O_2 = 8`.
theorem figurate_octagonal_two {
    figurate_octagonal(Nat.2) = Nat.8
} by {
    figurate_octagonal(Nat.2) = Nat.2 * (Nat.3 * Nat.2 - Nat.2)
    Nat.3 * Nat.2 = Nat.6
    Nat.6 - Nat.2 = Nat.4
    Nat.2 * Nat.4 = Nat.8
}

/// `O_3 = 21`.
theorem figurate_octagonal_three {
    figurate_octagonal(Nat.3) = Nat.21
} by {
    figurate_octagonal(Nat.3) = Nat.3 * (Nat.3 * Nat.3 - Nat.2)
    Nat.3 * Nat.3 = Nat.9
    Nat.9 - Nat.2 = Nat.7
    Nat.3 * Nat.7 = Nat.21
}
