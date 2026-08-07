from nat import Nat
from nat import mul_to_zero, add_to_zero, mul_one_right, alt_suc_ne_zero, mul_to_one,
    lte_mul_both
from number_theory.coprime import coprime_comm
from nat import exp_zero, exp_ne_zero
from nat import pow_distrib_mul
from data.basic.relation_transport import preserves_binary_op
numerals Nat

/// The constant arithmetic function with value one.
let nat_one_arithmetic_fn: Nat -> Nat = function(n: Nat) { Nat.1 }

/// The identity arithmetic function.
let nat_identity_arithmetic_fn: Nat -> Nat = function(n: Nat) { n }

/// The power arithmetic function `n -> n^k`.
define nat_power_arithmetic_fn(k: Nat) -> (Nat -> Nat) {
    function(n: Nat) { n.pow(k) }
}

/// The pointwise product of two arithmetic functions.
define arithmetic_fn_mul(f: Nat -> Nat, g: Nat -> Nat) -> (Nat -> Nat) {
    function(n: Nat) { f(n) * g(n) }
}

/// True if an arithmetic function is multiplicative on coprime arguments.
define is_multiplicative_nat_fn(f: Nat -> Nat) -> Bool {
    f(Nat.1) = Nat.1 and forall(a: Nat, b: Nat) {
        a.coprime(b) implies f(a * b) = f(a) * f(b)
    }
}

/// True if an arithmetic function is multiplicative on all arguments.
define is_completely_multiplicative_nat_fn(f: Nat -> Nat) -> Bool {
    f(Nat.1) = Nat.1 and forall(a: Nat, b: Nat) {
        f(a * b) = f(a) * f(b)
    }
}

/// A completely multiplicative arithmetic function is multiplicative.
theorem completely_multiplicative_imp_multiplicative(f: Nat -> Nat) {
    is_completely_multiplicative_nat_fn(f) implies is_multiplicative_nat_fn(f)
} by {
    if is_completely_multiplicative_nat_fn(f) {
        f(Nat.1) = Nat.1
        forall(a: Nat, b: Nat) {
            if a.coprime(b) {
                f(a * b) = f(a) * f(b)
            }
        }
    }
}

/// A completely multiplicative arithmetic function preserves one product.
theorem completely_multiplicative_nat_fn_apply(f: Nat -> Nat, a: Nat, b: Nat) {
    is_completely_multiplicative_nat_fn(f) implies f(a * b) = f(a) * f(b)
} by {
    if is_completely_multiplicative_nat_fn(f) {
        is_completely_multiplicative_nat_fn(f) = (f(Nat.1) = Nat.1 and forall(x: Nat, y: Nat) {
            f(x * y) = f(x) * f(y)
        })
        f(a * b) = f(a) * f(b)
    }
}

/// A multiplicative arithmetic function preserves one coprime product.
theorem multiplicative_nat_fn_apply(f: Nat -> Nat, a: Nat, b: Nat) {
    is_multiplicative_nat_fn(f) and a.coprime(b) implies f(a * b) = f(a) * f(b)
} by {
    if is_multiplicative_nat_fn(f) and a.coprime(b) {
        is_multiplicative_nat_fn(f) = (f(Nat.1) = Nat.1 and forall(x: Nat, y: Nat) {
            x.coprime(y) implies f(x * y) = f(x) * f(y)
        })
        f(a * b) = f(a) * f(b)
    }
}

/// The middle two factors in a product of four natural numbers may be swapped.
theorem nat_mul_swap_middle(a: Nat, b: Nat, c: Nat, d: Nat) {
    (a * b) * (c * d) = (a * c) * (b * d)
} by {
    (c * b) * d = c * (b * d)
    a * (b * (c * d)) = a * (c * (b * d))
}

/// The constant-one arithmetic function is completely multiplicative.
theorem nat_one_arithmetic_fn_completely_multiplicative {
    is_completely_multiplicative_nat_fn(nat_one_arithmetic_fn)
} by {
    forall(a: Nat, b: Nat) {
        nat_one_arithmetic_fn(a * b) =
            nat_one_arithmetic_fn(a) * nat_one_arithmetic_fn(b)
    }
}

/// The identity arithmetic function is completely multiplicative.
theorem nat_identity_arithmetic_fn_completely_multiplicative {
    is_completely_multiplicative_nat_fn(nat_identity_arithmetic_fn)
} by {
    forall(a: Nat, b: Nat) {
        nat_identity_arithmetic_fn(a * b) =
            nat_identity_arithmetic_fn(a) * nat_identity_arithmetic_fn(b)
    }
}

/// A natural-power arithmetic function is completely multiplicative.
theorem nat_power_arithmetic_fn_completely_multiplicative(k: Nat) {
    is_completely_multiplicative_nat_fn(nat_power_arithmetic_fn(k))
} by {
    exp_zero(Nat.1)
    nat_power_arithmetic_fn(k)(Nat.1) = Nat.1
    forall(a: Nat, b: Nat) {
        pow_distrib_mul[Nat](a, b, k)
        nat_power_arithmetic_fn(k)(a * b) =
            nat_power_arithmetic_fn(k)(a) * nat_power_arithmetic_fn(k)(b)
    }
}

/// The pointwise product of two completely multiplicative arithmetic functions
/// preserves one product.
theorem arithmetic_fn_mul_completely_multiplicative_apply(f: Nat -> Nat, g: Nat -> Nat,
    a: Nat, b: Nat) {
    is_completely_multiplicative_nat_fn(f) and is_completely_multiplicative_nat_fn(g)
        implies arithmetic_fn_mul(f, g)(a * b) =
            arithmetic_fn_mul(f, g)(a) * arithmetic_fn_mul(f, g)(b)
} by {
    if is_completely_multiplicative_nat_fn(f) and is_completely_multiplicative_nat_fn(g) {
        completely_multiplicative_nat_fn_apply(f, a, b)
        completely_multiplicative_nat_fn_apply(g, a, b)
        f(a * b) = f(a) * f(b)
        g(a * b) = g(a) * g(b)
        nat_mul_swap_middle(f(a), f(b), g(a), g(b))
        arithmetic_fn_mul(f, g)(a * b) =
            arithmetic_fn_mul(f, g)(a) * arithmetic_fn_mul(f, g)(b)
    }
}

/// The pointwise product of two multiplicative arithmetic functions preserves
/// one coprime product.
theorem arithmetic_fn_mul_multiplicative_apply(f: Nat -> Nat, g: Nat -> Nat,
    a: Nat, b: Nat) {
    is_multiplicative_nat_fn(f) and is_multiplicative_nat_fn(g)
        and a.coprime(b) implies arithmetic_fn_mul(f, g)(a * b) =
            arithmetic_fn_mul(f, g)(a) * arithmetic_fn_mul(f, g)(b)
} by {
    if is_multiplicative_nat_fn(f) and is_multiplicative_nat_fn(g) and a.coprime(b) {
        multiplicative_nat_fn_apply(f, a, b)
        multiplicative_nat_fn_apply(g, a, b)
        f(a * b) = f(a) * f(b)
        g(a * b) = g(a) * g(b)
        nat_mul_swap_middle(f(a), f(b), g(a), g(b))
        arithmetic_fn_mul(f, g)(a * b) =
            arithmetic_fn_mul(f, g)(a) * arithmetic_fn_mul(f, g)(b)
    }
}

/// The pointwise product of completely multiplicative arithmetic functions is
/// completely multiplicative.
theorem arithmetic_fn_mul_completely_multiplicative(f: Nat -> Nat, g: Nat -> Nat) {
    is_completely_multiplicative_nat_fn(f) and is_completely_multiplicative_nat_fn(g)
        implies is_completely_multiplicative_nat_fn(arithmetic_fn_mul(f, g))
} by {
    if is_completely_multiplicative_nat_fn(f) and is_completely_multiplicative_nat_fn(g) {
        is_completely_multiplicative_nat_fn(f) = (f(Nat.1) = Nat.1 and forall(a: Nat, b: Nat) {
            f(a * b) = f(a) * f(b)
        })
        is_completely_multiplicative_nat_fn(g) = (g(Nat.1) = Nat.1 and forall(a: Nat, b: Nat) {
            g(a * b) = g(a) * g(b)
        })
        arithmetic_fn_mul(f, g)(Nat.1) = Nat.1
        forall(a: Nat, b: Nat) {
            nat_mul_swap_middle(f(a), f(b), g(a), g(b))
            arithmetic_fn_mul(f, g)(a * b) =
                arithmetic_fn_mul(f, g)(a) * arithmetic_fn_mul(f, g)(b)
        }
        is_completely_multiplicative_nat_fn(arithmetic_fn_mul(f, g))
    }
}

/// The pointwise product of multiplicative arithmetic functions is
/// multiplicative.
theorem arithmetic_fn_mul_multiplicative(f: Nat -> Nat, g: Nat -> Nat) {
    is_multiplicative_nat_fn(f) and is_multiplicative_nat_fn(g)
        implies is_multiplicative_nat_fn(arithmetic_fn_mul(f, g))
} by {
    if is_multiplicative_nat_fn(f) and is_multiplicative_nat_fn(g) {
        is_multiplicative_nat_fn(f) = (f(Nat.1) = Nat.1 and forall(a: Nat, b: Nat) {
            a.coprime(b) implies f(a * b) = f(a) * f(b)
        })
        is_multiplicative_nat_fn(g) = (g(Nat.1) = Nat.1 and forall(a: Nat, b: Nat) {
            a.coprime(b) implies g(a * b) = g(a) * g(b)
        })
        arithmetic_fn_mul(f, g)(Nat.1) = Nat.1
        forall(a: Nat, b: Nat) {
            if a.coprime(b) {
                f(a * b) = f(a) * f(b)
                g(a * b) = g(a) * g(b)
                nat_mul_swap_middle(f(a), f(b), g(a), g(b))
                arithmetic_fn_mul(f, g)(a * b) =
                    arithmetic_fn_mul(f, g)(a) * arithmetic_fn_mul(f, g)(b)
            }
        }
        is_multiplicative_nat_fn(arithmetic_fn_mul(f, g))
    }
}

/// The constant arithmetic function with value zero.
let nat_zero_arithmetic_fn: Nat -> Nat = function(n: Nat) { Nat.0 }

/// The pointwise sum of two arithmetic functions.
define arithmetic_fn_add(f: Nat -> Nat, g: Nat -> Nat) -> (Nat -> Nat) {
    function(n: Nat) { f(n) + g(n) }
}

/// Application of a pointwise sum.
theorem arithmetic_fn_add_apply(f: Nat -> Nat, g: Nat -> Nat, n: Nat) {
    arithmetic_fn_add(f, g)(n) = f(n) + g(n)
}

/// Pointwise sum is commutative.
theorem arithmetic_fn_add_comm(f: Nat -> Nat, g: Nat -> Nat) {
    arithmetic_fn_add(f, g) = arithmetic_fn_add(g, f)
} by {
    forall(n: Nat) {
        f(n) + g(n) = g(n) + f(n)
        arithmetic_fn_add(f, g)(n) = arithmetic_fn_add(g, f)(n)
    }
}

/// Application of a pointwise product.
theorem arithmetic_fn_mul_apply(f: Nat -> Nat, g: Nat -> Nat, n: Nat) {
    arithmetic_fn_mul(f, g)(n) = f(n) * g(n)
}

/// Pointwise product is commutative.
theorem arithmetic_fn_mul_comm(f: Nat -> Nat, g: Nat -> Nat) {
    arithmetic_fn_mul(f, g) = arithmetic_fn_mul(g, f)
} by {
    forall(n: Nat) {
        f(n) * g(n) = g(n) * f(n)
        arithmetic_fn_mul(f, g)(n) = arithmetic_fn_mul(g, f)(n)
    }
}

/// The Dirichlet identity arithmetic function: 1 at n = 1 and 0 elsewhere.
define nat_dirichlet_unit_fn(n: Nat) -> Nat {
    if n = Nat.1 { Nat.1 } else { Nat.0 }
}

/// The Dirichlet identity sends 1 to 1.
theorem nat_dirichlet_unit_fn_at_one {
    nat_dirichlet_unit_fn(Nat.1) = Nat.1
}

/// The Dirichlet identity vanishes off 1.
theorem nat_dirichlet_unit_fn_off_one(n: Nat) {
    n != Nat.1 implies nat_dirichlet_unit_fn(n) = Nat.0
}

/// In the natural numbers, a product equals 1 only if both factors equal 1.
theorem nat_mul_eq_one(a: Nat, b: Nat) {
    a * b = Nat.1 implies a = Nat.1 and b = Nat.1
} by {
    if a * b = Nat.1 {
        if a = Nat.0 {
            Nat.0 = Nat.1
            false
        }
        if b = Nat.0 {
            a * Nat.0 = Nat.0
            false
        }
        a != Nat.0
        b != Nat.0
        if a != Nat.1 {
            Nat.1 < a
            b <= a * b
            // a >= 2 so a * b >= 2 * b >= 2 when b >= 1
            false
        }
        b = Nat.1
    }
}

/// The Dirichlet identity is completely multiplicative.
theorem nat_dirichlet_unit_fn_completely_multiplicative {
    is_completely_multiplicative_nat_fn(nat_dirichlet_unit_fn)
} by {
    nat_dirichlet_unit_fn(Nat.1) = Nat.1
    forall(a: Nat, b: Nat) {
        if a * b = Nat.1 {
            nat_mul_eq_one(a, b)
            nat_dirichlet_unit_fn(a * b) =
                nat_dirichlet_unit_fn(a) * nat_dirichlet_unit_fn(b)
        }
        if a * b != Nat.1 {
            nat_dirichlet_unit_fn(a * b) = Nat.0
            if a = Nat.1 and b = Nat.1 {
                false
            }
            a != Nat.1 or b != Nat.1
            nat_dirichlet_unit_fn(a) = Nat.0 or nat_dirichlet_unit_fn(b) = Nat.0
            nat_dirichlet_unit_fn(a) * nat_dirichlet_unit_fn(b) = Nat.0
            nat_dirichlet_unit_fn(a * b) =
                nat_dirichlet_unit_fn(a) * nat_dirichlet_unit_fn(b)
        }
        nat_dirichlet_unit_fn(a * b) =
            nat_dirichlet_unit_fn(a) * nat_dirichlet_unit_fn(b)
    }
}

/// The Dirichlet identity is multiplicative.
theorem nat_dirichlet_unit_fn_multiplicative {
    is_multiplicative_nat_fn(nat_dirichlet_unit_fn)
} by {
    nat_dirichlet_unit_fn_completely_multiplicative
    completely_multiplicative_imp_multiplicative(nat_dirichlet_unit_fn)
}

/// The constant-one arithmetic function is multiplicative.
theorem nat_one_arithmetic_fn_multiplicative {
    is_multiplicative_nat_fn(nat_one_arithmetic_fn)
} by {
    nat_one_arithmetic_fn_completely_multiplicative
    completely_multiplicative_imp_multiplicative(nat_one_arithmetic_fn)
}

/// The identity arithmetic function is multiplicative.
theorem nat_identity_arithmetic_fn_multiplicative {
    is_multiplicative_nat_fn(nat_identity_arithmetic_fn)
} by {
    nat_identity_arithmetic_fn_completely_multiplicative
    completely_multiplicative_imp_multiplicative(nat_identity_arithmetic_fn)
}

/// A natural-power arithmetic function is multiplicative.
theorem nat_power_arithmetic_fn_multiplicative(k: Nat) {
    is_multiplicative_nat_fn(nat_power_arithmetic_fn(k))
} by {
    nat_power_arithmetic_fn_completely_multiplicative(k)
    completely_multiplicative_imp_multiplicative(nat_power_arithmetic_fn(k))
}

/// Pointwise addition of arithmetic functions is associative.
theorem arithmetic_fn_add_assoc(f: Nat -> Nat, g: Nat -> Nat, h: Nat -> Nat) {
    arithmetic_fn_add(arithmetic_fn_add(f, g), h) =
        arithmetic_fn_add(f, arithmetic_fn_add(g, h))
} by {
    forall(n: Nat) {
        (f(n) + g(n)) + h(n) = f(n) + (g(n) + h(n))
        arithmetic_fn_add(arithmetic_fn_add(f, g), h)(n) =
            arithmetic_fn_add(f, arithmetic_fn_add(g, h))(n)
    }
}

/// Pointwise multiplication of arithmetic functions is associative.
theorem arithmetic_fn_mul_assoc(f: Nat -> Nat, g: Nat -> Nat, h: Nat -> Nat) {
    arithmetic_fn_mul(arithmetic_fn_mul(f, g), h) =
        arithmetic_fn_mul(f, arithmetic_fn_mul(g, h))
} by {
    forall(n: Nat) {
        (f(n) * g(n)) * h(n) = f(n) * (g(n) * h(n))
        arithmetic_fn_mul(arithmetic_fn_mul(f, g), h)(n) =
            arithmetic_fn_mul(f, arithmetic_fn_mul(g, h))(n)
    }
}

/// Application of the constant-zero arithmetic function.
theorem nat_zero_arithmetic_fn_apply(n: Nat) {
    nat_zero_arithmetic_fn(n) = Nat.0
}

/// Application of the constant-one arithmetic function.
theorem nat_one_arithmetic_fn_apply(n: Nat) {
    nat_one_arithmetic_fn(n) = Nat.1
}

/// Application of the identity arithmetic function.
theorem nat_identity_arithmetic_fn_apply(n: Nat) {
    nat_identity_arithmetic_fn(n) = n
}

/// Application of a natural-power arithmetic function.
theorem nat_power_arithmetic_fn_apply(k: Nat, n: Nat) {
    nat_power_arithmetic_fn(k)(n) = n.pow(k)
}

/// The zero arithmetic function is a left identity for pointwise addition.
theorem arithmetic_fn_add_zero_left(f: Nat -> Nat) {
    arithmetic_fn_add(nat_zero_arithmetic_fn, f) = f
} by {
    forall(n: Nat) {
        arithmetic_fn_add(nat_zero_arithmetic_fn, f)(n) = f(n)
    }
}

/// The zero arithmetic function is a right identity for pointwise addition.
theorem arithmetic_fn_add_zero_right(f: Nat -> Nat) {
    arithmetic_fn_add(f, nat_zero_arithmetic_fn) = f
} by {
    arithmetic_fn_add_comm(f, nat_zero_arithmetic_fn)
    arithmetic_fn_add_zero_left(f)
}

/// The constant-one arithmetic function is a left identity for pointwise
/// multiplication.
theorem arithmetic_fn_mul_one_left(f: Nat -> Nat) {
    arithmetic_fn_mul(nat_one_arithmetic_fn, f) = f
} by {
    forall(n: Nat) {
        arithmetic_fn_mul(nat_one_arithmetic_fn, f)(n) = f(n)
    }
}

/// The constant-one arithmetic function is a right identity for pointwise
/// multiplication.
theorem arithmetic_fn_mul_one_right(f: Nat -> Nat) {
    arithmetic_fn_mul(f, nat_one_arithmetic_fn) = f
} by {
    arithmetic_fn_mul_comm(f, nat_one_arithmetic_fn)
    arithmetic_fn_mul_one_left(f)
}

/// The constant-zero arithmetic function annihilates pointwise multiplication
/// from the left.
theorem arithmetic_fn_mul_zero_left(f: Nat -> Nat) {
    arithmetic_fn_mul(nat_zero_arithmetic_fn, f) = nat_zero_arithmetic_fn
} by {
    forall(n: Nat) {
        arithmetic_fn_mul(nat_zero_arithmetic_fn, f)(n) = Nat.0
        arithmetic_fn_mul(nat_zero_arithmetic_fn, f)(n) =
            nat_zero_arithmetic_fn(n)
    }
}

/// The constant-zero arithmetic function annihilates pointwise multiplication
/// from the right.
theorem arithmetic_fn_mul_zero_right(f: Nat -> Nat) {
    arithmetic_fn_mul(f, nat_zero_arithmetic_fn) = nat_zero_arithmetic_fn
} by {
    arithmetic_fn_mul_comm(f, nat_zero_arithmetic_fn)
    arithmetic_fn_mul_zero_left(f)
}

/// True if `f(n)` is positive for every positive `n`. A handy hypothesis when
/// arguing about multiplicativity over coprime products of positive arguments.
define positive_on_positive(f: Nat -> Nat) -> Bool {
    forall(n: Nat) { Nat.0 < n implies Nat.0 < f(n) }
}

/// The constant-one arithmetic function is positive on positive arguments.
theorem nat_one_arithmetic_fn_positive_on_positive {
    positive_on_positive(nat_one_arithmetic_fn)
} by {
    forall(n: Nat) {
        if Nat.0 < n {
            Nat.0 < Nat.1
            Nat.0 < nat_one_arithmetic_fn(n)
        }
    }
}

/// The identity arithmetic function is positive on positive arguments.
theorem nat_identity_arithmetic_fn_positive_on_positive {
    positive_on_positive(nat_identity_arithmetic_fn)
} by {
    forall(n: Nat) {
        if Nat.0 < n {
            Nat.0 < nat_identity_arithmetic_fn(n)
        }
    }
}

/// A natural-power arithmetic function is positive on positive arguments.
theorem nat_power_arithmetic_fn_positive_on_positive(k: Nat) {
    positive_on_positive(nat_power_arithmetic_fn(k))
} by {
    forall(n: Nat) {
        if Nat.0 < n {
            exp_ne_zero(n, k)
            n.pow(k) != Nat.0
            Nat.0 < nat_power_arithmetic_fn(k)(n)
        }
    }
}

/// A multiplicative arithmetic function takes the value one at one.
theorem is_multiplicative_nat_fn_at_one(f: Nat -> Nat) {
    is_multiplicative_nat_fn(f) implies f(Nat.1) = Nat.1
} by {
    if is_multiplicative_nat_fn(f) {
        f(Nat.1) = Nat.1
    }
}

/// A completely multiplicative arithmetic function takes the value one at one.
theorem is_completely_multiplicative_nat_fn_at_one(f: Nat -> Nat) {
    is_completely_multiplicative_nat_fn(f) implies f(Nat.1) = Nat.1
} by {
    if is_completely_multiplicative_nat_fn(f) {
        f(Nat.1) = Nat.1
    }
}

/// Pointwise sum with a left summand positive on positive arguments is itself positive on
/// positive arguments.
theorem arithmetic_fn_add_positive_on_positive_left(f: Nat -> Nat, g: Nat -> Nat) {
    positive_on_positive(f) implies positive_on_positive(arithmetic_fn_add(f, g))
} by {
    if positive_on_positive(f) {
        positive_on_positive(f) = forall(x: Nat) {
            Nat.0 < x implies Nat.0 < f(x)
        }
        forall(n: Nat) {
            if Nat.0 < n {
                if f(n) + g(n) = Nat.0 {
                    add_to_zero(f(n), g(n))
                    false
                }
                Nat.0 < arithmetic_fn_add(f, g)(n)
            }
        }
    }
}

/// Pointwise sum with a right summand positive on positive arguments is itself positive on
/// positive arguments.
theorem arithmetic_fn_add_positive_on_positive_right(f: Nat -> Nat, g: Nat -> Nat) {
    positive_on_positive(g) implies positive_on_positive(arithmetic_fn_add(f, g))
} by {
    if positive_on_positive(g) {
        arithmetic_fn_add_comm(f, g)
        arithmetic_fn_add_positive_on_positive_left(g, f)
        positive_on_positive(arithmetic_fn_add(g, f))
    }
}

/// Pointwise product preserves positivity on positive arguments.
theorem arithmetic_fn_mul_positive_on_positive(f: Nat -> Nat, g: Nat -> Nat) {
    positive_on_positive(f) and positive_on_positive(g)
        implies positive_on_positive(arithmetic_fn_mul(f, g))
} by {
    if positive_on_positive(f) and positive_on_positive(g) {
        positive_on_positive(f) = forall(x: Nat) {
            Nat.0 < x implies Nat.0 < f(x)
        }
        positive_on_positive(g) = forall(x: Nat) {
            Nat.0 < x implies Nat.0 < g(x)
        }
        forall(n: Nat) {
            if Nat.0 < n {
                Nat.0 < f(n)
                Nat.0 < g(n)
                arithmetic_fn_mul(f, g)(n) = f(n) * g(n)
                if f(n) * g(n) = Nat.0 {
                    mul_to_zero(f(n), g(n))
                    false
                }
                Nat.0 < arithmetic_fn_mul(f, g)(n)
            }
        }
    }
}
