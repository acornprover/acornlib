// The Liouville function `lambda(n) = (-1)^Omega(n)`, deepened.
//
// The basic identities for the Liouville function:
//
//   1. `lambda(p) = -1` for every prime `p`, since a prime has exactly one
//      prime factor (Section 1).
//   2. `lambda(p^k) = (-1)^k` for every prime `p` and exponent `k`, since
//      `Omega(p^k) = k` (Section 2).
//   3. Complete multiplicativity on the positive naturals:
//      `lambda(mn) = lambda(m) * lambda(n)` for `m, n > 0` (Section 3).
//      The placeholder value `lambda(0) = 1` breaks the identity at zero:
//      `lambda(0 * 2) = 1` but `lambda(0) * lambda(2) = -1`, recorded below.
//   4. The parity characterisation: `lambda(n) = 1` exactly when `Omega(n)`
//      is even, since the alternating sign is one at even indices and minus
//      one at odd indices (Section 4).
//   5. The connection to the Mobius function:
//      `mu(n) = lambda(n) * |mu(n)|` for every natural `n`; that is, `mu`
//      agrees with `lambda` on squarefree numbers and both sides vanish
//      elsewhere (Section 5).
from nat import Nat, pos_of_ne_zero, lt_suc, from_nat_one, from_nat_zero
from int import Int, abs, abs_neg, abs_from_nat, mul_one_right, mul_zero_right
from list import List
from algebra.ring.ring import Ring, alternating_sign, alternating_sign_zero,
    alternating_sign_suc, alternating_sign_eq_neg_one_pow, alternating_sign_parity
from number_theory.liouville import nat_liouville, nat_prime_omega, nat_liouville_mul,
    nat_liouville_one_or_neg_one, nat_liouville_two, nat_liouville_four,
    nat_liouville_six, nat_two_prime_local, nat_three_prime
from number_theory.omega_omega_big import nat_prime_omega_prime, nat_prime_omega_pow
from number_theory.factorisation import prime_factorisation, prime_factorisation_zero
from number_theory.mobius_inversion import nat_mobius, nat_mobius_zero,
    nat_mobius_prime, nat_mobius_multiplicative_apply
from number_theory.totient import coprime_below_prime
from data.nat.nat_squarefree import is_squarefree
from number_theory.squarefree import zero_not_squarefree,
    is_squarefree_iff_factorisation_unique
numerals Nat
numerals Int

// ---------------------------------------------------------------------------
// Section 1: values at zero and at primes
// ---------------------------------------------------------------------------

/// `lambda(0) = 1`: the placeholder factorisation of zero is empty, so zero
/// has no prime factors and `lambda(0) = (-1)^0 = 1`.
theorem nat_liouville_zero {
    nat_liouville(Nat.0) = Int.1
} by {
    prime_factorisation_zero
    prime_factorisation(Nat.0) = List.nil[Nat]
    List.nil[Nat].length = Nat.0
    alternating_sign_zero[Int]
    alternating_sign[Int](Nat.0) = Int.1
    nat_prime_omega(Nat.0) = prime_factorisation(Nat.0).length
    nat_prime_omega(Nat.0) = Nat.0
    alternating_sign[Int](nat_prime_omega(Nat.0)) = alternating_sign[Int](Nat.0)
    alternating_sign[Int](nat_prime_omega(Nat.0)) = Int.1
    nat_liouville(Nat.0) = alternating_sign[Int](nat_prime_omega(Nat.0))
    nat_liouville(Nat.0) = Int.1
}

/// `lambda(p) = -1` for every prime `p`: a prime has a single prime factor,
/// so `Omega(p) = 1` and `lambda(p) = (-1)^1 = -1`.
theorem nat_liouville_prime(p: Nat) {
    p.is_prime implies nat_liouville(p) = -Int.1
} by {
    if p.is_prime {
        nat_prime_omega_prime(p)
        nat_prime_omega(p) = Nat.1
        alternating_sign_suc[Int](Nat.0)
        alternating_sign[Int](Nat.0.suc) = -alternating_sign[Int](Nat.0)
        alternating_sign_zero[Int]
        alternating_sign[Int](Nat.0) = Int.1
        Nat.0.suc = Nat.1
        alternating_sign[Int](Nat.1) = -Int.1
        alternating_sign[Int](nat_prime_omega(p)) = alternating_sign[Int](Nat.1)
        alternating_sign[Int](nat_prime_omega(p)) = -Int.1
        nat_liouville(p) = alternating_sign[Int](nat_prime_omega(p))
        nat_liouville(p) = -Int.1
    }
}

// ---------------------------------------------------------------------------
// Section 2: values at prime powers
// ---------------------------------------------------------------------------

/// `lambda(p^k) = (-1)^k` for every prime `p` and exponent `k`: the prime
/// power `p^k` has exactly `k` prime factors counted with multiplicity, so
/// `Omega(p^k) = k` and `lambda(p^k) = (-1)^k`.
theorem nat_liouville_pow(p: Nat, k: Nat) {
    p.is_prime implies nat_liouville(p.pow(k)) = (-Int.1).pow(k)
} by {
    if p.is_prime {
        nat_prime_omega_pow(p, k)
        nat_prime_omega(p.pow(k)) = k
        alternating_sign_eq_neg_one_pow[Int](k)
        alternating_sign[Int](k) = (-Int.1).pow(k)
        nat_liouville(p.pow(k)) = alternating_sign[Int](nat_prime_omega(p.pow(k)))
        alternating_sign[Int](nat_prime_omega(p.pow(k))) = alternating_sign[Int](k)
        alternating_sign[Int](nat_prime_omega(p.pow(k))) = (-Int.1).pow(k)
        nat_liouville(p.pow(k)) = (-Int.1).pow(k)
    }
}

// ---------------------------------------------------------------------------
// Section 3: complete multiplicativity on the positive naturals
// ---------------------------------------------------------------------------

/// Complete multiplicativity fails at zero: `lambda(0 * 2) = lambda(0) = 1`
/// but `lambda(0) * lambda(2) = 1 * -1 = -1`.
theorem nat_liouville_mul_fails_at_zero {
    not (nat_liouville(Nat.0 * Nat.2) = nat_liouville(Nat.0) * nat_liouville(Nat.2))
} by {
    if nat_liouville(Nat.0 * Nat.2) = nat_liouville(Nat.0) * nat_liouville(Nat.2) {
        nat_liouville_zero
        nat_liouville(Nat.0) = Int.1
        nat_liouville_two
        nat_liouville(Nat.2) = -Int.1
        Nat.0 * Nat.2 = Nat.0
        nat_liouville(Nat.0 * Nat.2) = nat_liouville(Nat.0)
        nat_liouville(Nat.0 * Nat.2) = Int.1
        nat_liouville(Nat.0) * nat_liouville(Nat.2) = Int.1 * -Int.1
        Int.1 * -Int.1 = -Int.1
        nat_liouville(Nat.0) * nat_liouville(Nat.2) = -Int.1
        nat_liouville(Nat.0 * Nat.2) = -Int.1
        Int.1 = -Int.1
        false
    }
}

// ---------------------------------------------------------------------------
// Section 4: the parity characterisation
// ---------------------------------------------------------------------------

/// The alternating sign is minus one at odd indices.
theorem nat_alternating_sign_odd_neg_one(k: Nat) {
    not (Nat.2.divides(k)) implies alternating_sign[Int](k) = -Int.1
} by {
    alternating_sign_parity[Int](k)
    not (Nat.2.divides(k)) implies alternating_sign[Int](k) = -Int.1
}

/// The alternating sign is one at even indices.
theorem nat_alternating_sign_even_one(k: Nat) {
    Nat.2.divides(k) implies alternating_sign[Int](k) = Int.1
} by {
    alternating_sign_parity[Int](k)
    Nat.2.divides(k) implies alternating_sign[Int](k) = Int.1
}

/// The parity characterisation: `lambda(n) = 1` exactly when `Omega(n)` is
/// even, since the alternating sign is one at even indices and minus one at
/// odd indices.
theorem nat_liouville_one_iff_omega_even(n: Nat) {
    (nat_liouville(n) = Int.1) = (Nat.2.divides(nat_prime_omega(n)))
} by {
    if nat_liouville(n) = Int.1 {
        if Nat.2.divides(nat_prime_omega(n)) {
        } else {
            nat_alternating_sign_odd_neg_one(nat_prime_omega(n))
            alternating_sign[Int](nat_prime_omega(n)) = -Int.1
            nat_liouville(n) = alternating_sign[Int](nat_prime_omega(n))
            nat_liouville(n) = -Int.1
            nat_liouville(n) = Int.1
            Int.1 = -Int.1
            false
        }
    }
    if Nat.2.divides(nat_prime_omega(n)) {
        nat_alternating_sign_even_one(nat_prime_omega(n))
        alternating_sign[Int](nat_prime_omega(n)) = Int.1
        nat_liouville(n) = alternating_sign[Int](nat_prime_omega(n))
        nat_liouville(n) = Int.1
    }
}

/// `lambda(4) = 1` and `Omega(4)` is even, matching the parity
/// characterisation at the square of two.
theorem nat_liouville_four_omega_even {
    (nat_liouville(Nat.4) = Int.1) and Nat.2.divides(nat_prime_omega(Nat.4))
} by {
    nat_liouville_four
    nat_liouville(Nat.4) = Int.1
    nat_liouville_one_iff_omega_even(Nat.4)
    (nat_liouville(Nat.4) = Int.1) = (Nat.2.divides(nat_prime_omega(Nat.4)))
    Nat.2.divides(nat_prime_omega(Nat.4))
}

// ---------------------------------------------------------------------------
// Section 5: the connection to the Mobius function
// ---------------------------------------------------------------------------

/// The absolute value of the Liouville function is one everywhere.
theorem abs_nat_liouville_one(n: Nat) {
    abs(nat_liouville(n)) = Nat.1
} by {
    nat_liouville_one_or_neg_one(n)
    if nat_liouville(n) = Int.1 {
        abs(nat_liouville(n)) = abs(Int.1)
        abs_from_nat(Nat.1)
        abs(Int.from_nat(Nat.1)) = Nat.1
        from_nat_one[Int]
        Int.from_nat(Nat.1) = Int.1
        abs(Int.1) = Nat.1
        abs(nat_liouville(n)) = Nat.1
    }
    if nat_liouville(n) = -Int.1 {
        abs(nat_liouville(n)) = abs(-Int.1)
        abs_neg(Int.1)
        abs(-Int.1) = abs(Int.1)
        abs(Int.1) = Nat.1
        abs(-Int.1) = Nat.1
        abs(nat_liouville(n)) = Nat.1
    }
}

/// `mu` agrees with `lambda` on squarefree numbers: when the prime
/// factorisation of `n` has no repeated prime, both functions are
/// `(-1)^k` for `k` the number of prime factors.
theorem nat_mobius_eq_liouville_factorisation_unique(n: Nat) {
    n != Nat.0 and prime_factorisation(n).is_unique implies nat_mobius(n) = nat_liouville(n)
} by {
    if n != Nat.0 and prime_factorisation(n).is_unique {
        n != Nat.0
        prime_factorisation(n).is_unique
        nat_mobius(n) = alternating_sign[Int](prime_factorisation(n).length)
        nat_prime_omega(n) = prime_factorisation(n).length
        nat_liouville(n) = alternating_sign[Int](nat_prime_omega(n))
        nat_liouville(n) = alternating_sign[Int](prime_factorisation(n).length)
        nat_mobius(n) = nat_liouville(n)
    }
}

/// `mu` agrees with `lambda` on squarefree numbers.
theorem nat_mobius_eq_liouville_squarefree(n: Nat) {
    is_squarefree(n) implies nat_mobius(n) = nat_liouville(n)
} by {
    if is_squarefree(n) {
        if n = Nat.0 {
            zero_not_squarefree
            not is_squarefree(Nat.0)
            is_squarefree(n) = is_squarefree(Nat.0)
            false
        } else {
            n != Nat.0
            pos_of_ne_zero(n)
            Nat.0 < n
            is_squarefree_iff_factorisation_unique(n)
            (is_squarefree(n) = prime_factorisation(n).is_unique)
            prime_factorisation(n).is_unique
            nat_mobius_eq_liouville_factorisation_unique(n)
            nat_mobius(n) = nat_liouville(n)
        }
    }
}

/// The connection to the Mobius function:
/// `mu(n) = lambda(n) * |mu(n)|` for every natural `n`: on squarefree `n`
/// the Mobius value is `lambda(n) = +/-1` with absolute value one, and
/// elsewhere both sides vanish.
theorem nat_mobius_eq_liouville_mul_abs(n: Nat) {
    nat_mobius(n) = nat_liouville(n) * Int.from_nat(abs(nat_mobius(n)))
} by {
    if n = Nat.0 {
        nat_mobius_zero
        nat_mobius(Nat.0) = Int.0
        nat_mobius(n) = Int.0
        abs(nat_mobius(n)) = abs(Int.0)
        abs_from_nat(Nat.0)
        abs(Int.from_nat(Nat.0)) = Nat.0
        from_nat_zero[Int]
        Int.from_nat(Nat.0) = Int.0
        abs(Int.0) = Nat.0
        abs(nat_mobius(n)) = Nat.0
        Int.from_nat(abs(nat_mobius(n))) = Int.from_nat(Nat.0)
        Int.from_nat(abs(nat_mobius(n))) = Int.0
        nat_liouville(n) * Int.from_nat(abs(nat_mobius(n))) =
            nat_liouville(n) * Int.0
        mul_zero_right(nat_liouville(n))
        nat_liouville(n) * Int.0 = Int.0
        nat_liouville(n) * Int.from_nat(abs(nat_mobius(n))) = Int.0
        nat_mobius(n) = nat_liouville(n) * Int.from_nat(abs(nat_mobius(n)))
    } else {
        n != Nat.0
        if prime_factorisation(n).is_unique {
            nat_mobius_eq_liouville_factorisation_unique(n)
            nat_mobius(n) = nat_liouville(n)
            abs(nat_mobius(n)) = abs(nat_liouville(n))
            abs_nat_liouville_one(n)
            abs(nat_liouville(n)) = Nat.1
            abs(nat_mobius(n)) = Nat.1
            Int.from_nat(abs(nat_mobius(n))) = Int.from_nat(Nat.1)
            from_nat_one[Int]
            Int.from_nat(Nat.1) = Int.1
            Int.from_nat(abs(nat_mobius(n))) = Int.1
            nat_liouville(n) * Int.from_nat(abs(nat_mobius(n))) =
                nat_liouville(n) * Int.1
            mul_one_right(nat_liouville(n))
            nat_liouville(n) * Int.1 = nat_liouville(n)
            nat_liouville(n) * Int.from_nat(abs(nat_mobius(n))) = nat_liouville(n)
            nat_liouville(n) * Int.from_nat(abs(nat_mobius(n))) = nat_mobius(n)
            nat_mobius(n) = nat_liouville(n) * Int.from_nat(abs(nat_mobius(n)))
        } else {
            not prime_factorisation(n).is_unique
            nat_mobius(n) = Int.0
            abs(nat_mobius(n)) = abs(Int.0)
            abs(Int.0) = Nat.0
            abs(nat_mobius(n)) = Nat.0
            Int.from_nat(abs(nat_mobius(n))) = Int.from_nat(Nat.0)
            Int.from_nat(abs(nat_mobius(n))) = Int.0
            nat_liouville(n) * Int.from_nat(abs(nat_mobius(n))) =
                nat_liouville(n) * Int.0
            mul_zero_right(nat_liouville(n))
            nat_liouville(n) * Int.0 = Int.0
            nat_liouville(n) * Int.from_nat(abs(nat_mobius(n))) = Int.0
            nat_mobius(n) = nat_liouville(n) * Int.from_nat(abs(nat_mobius(n)))
        }
    }
}

/// `mu(6) = 1`: six is the product of the two distinct primes two and three.
theorem nat_mobius_six {
    nat_mobius(Nat.6) = Int.1
} by {
    nat_two_prime_local
    Nat.2.is_prime
    nat_three_prime
    Nat.3.is_prime
    Nat.1 <= Nat.2
    lt_suc(Nat.2)
    Nat.2 < Nat.3
    coprime_below_prime(Nat.3, Nat.2)
    Nat.2.coprime(Nat.3)
    nat_mobius_multiplicative_apply(Nat.2, Nat.3)
    nat_mobius(Nat.2 * Nat.3) = nat_mobius(Nat.2) * nat_mobius(Nat.3)
    Nat.2 * Nat.3 = Nat.6
    nat_mobius(Nat.6) = nat_mobius(Nat.2) * nat_mobius(Nat.3)
    nat_mobius_prime(Nat.2)
    nat_mobius(Nat.2) = -Int.1
    nat_mobius_prime(Nat.3)
    nat_mobius(Nat.3) = -Int.1
    -Int.1 * -Int.1 = Int.1
    nat_mobius(Nat.6) = Int.1
}

/// The connection at six: `mu(6) = 1 = lambda(6)`.
theorem nat_mobius_liouville_six {
    nat_mobius(Nat.6) = Int.1 and nat_liouville(Nat.6) = Int.1
} by {
    nat_mobius_six
    nat_mobius(Nat.6) = Int.1
    nat_liouville_six
    nat_liouville(Nat.6) = Int.1
}
