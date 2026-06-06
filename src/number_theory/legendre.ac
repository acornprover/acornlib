from nat import Nat, factorial_zero, factorial_step, lte_one_factorial, only_zero_lte_zero, mul_to_zero, lt_not_ref
from nat import zero_or_suc, lte_suc_suc, pow_one
from number_theory.factorisation import count_prime_factor, count_prime_factor_one, count_prime_factor_mul, prime_pow_divides_iff
from combinatorics import binom, choose_add, binom_pos
numerals Nat

/// The running total of the p-adic valuations of `1, 2, ..., n`, where the
/// valuation of `m` is `count_prime_factor(p, m)`. Legendre's identity equates
/// this with the p-adic valuation of `n!`.
define prime_factor_count_upto(p: Nat, n: Nat) -> Nat {
    match n {
        Nat.zero {
            Nat.0
        }
        Nat.suc(k) {
            prime_factor_count_upto(p, k) + count_prime_factor(p, n)
        }
    }
}

/// The empty initial segment contributes nothing.
theorem prime_factor_count_upto_zero(p: Nat) {
    prime_factor_count_upto(p, Nat.0) = Nat.0
}

/// Extending the range by one adds the valuation of the new top element.
theorem prime_factor_count_upto_step(p: Nat, n: Nat) {
    prime_factor_count_upto(p, n.suc) =
        prime_factor_count_upto(p, n) + count_prime_factor(p, n.suc)
}

/// The factorial is positive, hence nonzero.
theorem factorial_ne_zero(n: Nat) {
    n.factorial != Nat.0
} by {
    lte_one_factorial(n)
    if n.factorial = Nat.0 {
        Nat.1 <= Nat.0
        only_zero_lte_zero(Nat.1)
        false
    }
}

/// Legendre's identity in additive form: the p-adic valuation of `n!` is the
/// sum of the p-adic valuations of `1, ..., n`.
theorem legendre_factorial(p: Nat, n: Nat) {
    count_prime_factor(p, n.factorial) = prime_factor_count_upto(p, n)
} by {
    let g: Nat -> Bool = function(x: Nat) {
        count_prime_factor(p, x.factorial) = prime_factor_count_upto(p, x)
    }
    factorial_zero
    count_prime_factor_one(p)
    prime_factor_count_upto_zero(p)
    g(Nat.0)
    forall(x: Nat) {
        if g(x) {
            count_prime_factor(p, x.factorial) = prime_factor_count_upto(p, x)
            factorial_step(x)
            factorial_ne_zero(x)
            count_prime_factor_mul(p, x.suc, x.factorial)
            count_prime_factor(p, x.suc * x.factorial) =
                count_prime_factor(p, x.suc) + count_prime_factor(p, x.factorial)
            count_prime_factor(p, x.suc.factorial) =
                count_prime_factor(p, x.suc) + count_prime_factor(p, x.factorial)
            count_prime_factor(p, x.suc.factorial) =
                count_prime_factor(p, x.suc) + prime_factor_count_upto(p, x)
            prime_factor_count_upto_step(p, x)
            count_prime_factor(p, x.suc.factorial) = prime_factor_count_upto(p, x.suc)
            g(x.suc)
        }
    }
    g(n)
    count_prime_factor(p, n.factorial) = prime_factor_count_upto(p, n)
}

/// A product of nonzero naturals is nonzero.
theorem mul_ne_zero(a: Nat, b: Nat) {
    a != Nat.0 and b != Nat.0 implies a * b != Nat.0
} by {
    if a != Nat.0 and b != Nat.0 {
        if a * b = Nat.0 {
            mul_to_zero(a, b)
            false
        }
    }
}

/// The binomial coefficient `binom(a+b, a)` is nonzero.
theorem binom_add_ne_zero(a: Nat, b: Nat) {
    (a + b).binom(a) != Nat.0
} by {
    a <= a + b
    binom_pos(a + b, a)
    if (a + b).binom(a) = Nat.0 {
        Nat.0 < Nat.0
        lt_not_ref(Nat.0)
        false
    }
}

/// The p-adic valuation accounting for `binom(a+b,a)` in factorial form.
theorem legendre_binom_factorial(p: Nat, a: Nat, b: Nat) {
    count_prime_factor(p, (a + b).binom(a)) +
        count_prime_factor(p, a.factorial) +
        count_prime_factor(p, b.factorial) =
        count_prime_factor(p, (a + b).factorial)
} by {
    choose_add(a, b)
    binom_add_ne_zero(a, b)
    factorial_ne_zero(a)
    factorial_ne_zero(b)
    mul_ne_zero((a + b).binom(a), a.factorial)
    count_prime_factor_mul(p, (a + b).binom(a), a.factorial)
    count_prime_factor(p, (a + b).binom(a) * a.factorial) =
        count_prime_factor(p, (a + b).binom(a)) + count_prime_factor(p, a.factorial)
    count_prime_factor_mul(p, (a + b).binom(a) * a.factorial, b.factorial)
    count_prime_factor(p, (a + b).binom(a) * a.factorial * b.factorial) =
        count_prime_factor(p, (a + b).binom(a) * a.factorial) + count_prime_factor(p, b.factorial)
    count_prime_factor(p, (a + b).binom(a) * a.factorial * b.factorial) =
        count_prime_factor(p, (a + b).binom(a)) +
        count_prime_factor(p, a.factorial) +
        count_prime_factor(p, b.factorial)
    count_prime_factor(p, (a + b).factorial) =
        count_prime_factor(p, (a + b).binom(a)) +
        count_prime_factor(p, a.factorial) +
        count_prime_factor(p, b.factorial)
    count_prime_factor(p, (a + b).binom(a)) +
        count_prime_factor(p, a.factorial) +
        count_prime_factor(p, b.factorial) =
        count_prime_factor(p, (a + b).factorial)
}

/// Legendre's identity for binomial coefficients, additive form: the p-adic
/// valuation of `binom(a+b, a)` together with those of `a!` and `b!` accounts
/// for the valuation of `(a+b)!`.
theorem legendre_binom(p: Nat, a: Nat, b: Nat) {
    count_prime_factor(p, (a + b).binom(a)) +
        prime_factor_count_upto(p, a) +
        prime_factor_count_upto(p, b) =
        prime_factor_count_upto(p, a + b)
} by {
    choose_add(a, b)
    binom_add_ne_zero(a, b)
    factorial_ne_zero(a)
    factorial_ne_zero(b)
    mul_ne_zero((a + b).binom(a), a.factorial)
    count_prime_factor_mul(p, (a + b).binom(a), a.factorial)
    count_prime_factor(p, (a + b).binom(a) * a.factorial) =
        count_prime_factor(p, (a + b).binom(a)) + count_prime_factor(p, a.factorial)
    count_prime_factor_mul(p, (a + b).binom(a) * a.factorial, b.factorial)
    count_prime_factor(p, (a + b).binom(a) * a.factorial * b.factorial) =
        count_prime_factor(p, (a + b).binom(a) * a.factorial) + count_prime_factor(p, b.factorial)
    count_prime_factor(p, (a + b).binom(a) * a.factorial * b.factorial) =
        count_prime_factor(p, (a + b).binom(a)) +
        count_prime_factor(p, a.factorial) +
        count_prime_factor(p, b.factorial)
    count_prime_factor(p, (a + b).factorial) =
        count_prime_factor(p, (a + b).binom(a)) +
        count_prime_factor(p, a.factorial) +
        count_prime_factor(p, b.factorial)
    legendre_factorial(p, a + b)
    legendre_factorial(p, a)
    legendre_factorial(p, b)
    count_prime_factor(p, (a + b).binom(a)) +
        prime_factor_count_upto(p, a) +
        prime_factor_count_upto(p, b) =
        prime_factor_count_upto(p, a + b)
}

/// For naturals, being at least one is the same as being nonzero.
theorem one_lte_iff_ne_zero(x: Nat) {
    (Nat.1 <= x) = (x != Nat.0)
} by {
    if Nat.1 <= x {
        if x = Nat.0 {
            Nat.1 <= Nat.0
            only_zero_lte_zero(Nat.1)
            false
        }
        x != Nat.0
    }
    if x != Nat.0 {
        zero_or_suc(x)
        let y: Nat satisfy { x = y.suc }
        Nat.0 <= y
        lte_suc_suc(Nat.0, y)
        Nat.1 <= x
    }
    (Nat.1 <= x) = (x != Nat.0)
}

/// A prime `p` divides `n` (for nonzero `n`) exactly when its valuation
/// `count_prime_factor(p, n)` is nonzero. This bridges divisibility and the
/// additive valuation accounting above.
theorem prime_divides_iff_count_ne_zero(p: Nat, n: Nat) {
    p.is_prime and n != Nat.0 implies (
        p.divides(n) = (count_prime_factor(p, n) != Nat.0)
    )
} by {
    if p.is_prime and n != Nat.0 {
        prime_pow_divides_iff(p, Nat.1, n)
        pow_one(p)
        p.divides(n) = (Nat.1 <= count_prime_factor(p, n))
        one_lte_iff_ne_zero(count_prime_factor(p, n))
        p.divides(n) = (count_prime_factor(p, n) != Nat.0)
    }
}

/// A prime divides `binom(a+b, a)` exactly when its valuation there is nonzero.
/// Combined with `legendre_binom`, this turns coprimality questions about
/// binomial coefficients into valuation arithmetic.
theorem prime_divides_binom_iff(p: Nat, a: Nat, b: Nat) {
    p.is_prime implies (
        p.divides((a + b).binom(a)) =
            (count_prime_factor(p, (a + b).binom(a)) != Nat.0)
    )
} by {
    if p.is_prime {
        binom_add_ne_zero(a, b)
        prime_divides_iff_count_ne_zero(p, (a + b).binom(a))
    }
}
