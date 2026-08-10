/// Continued fractions and Pell's equation: the classical link between the
/// convergents p_n/q_n of the continued fraction of √2 and the solutions of
/// the Pell equation x² - 2·y² = 1.
///
/// The main result is the classical identity
///
///     p_n² - 2·q_n² = (-1)^(n+1),
///
/// proved by the recurrence: every convergent of √2 is obtained from the
/// previous one by composing with the fundamental unit 1 + √2, namely
///
///     p_{n+1} = p_n + 2·q_n,    q_{n+1} = p_n + q_n,
///
/// and the composition flips the sign of the norm x² - 2·y². Hence every
/// convergent has norm ±1; the odd-indexed convergents solve Pell's equation
/// x² - 2·y² = 1 exactly, and every convergent is either a Pell solution or
/// its successor is. This extends the d = 2 fundamental solution (3, 2) =
/// p_1/q_1 proved in pell.ac to the whole convergent sequence.
///
/// The general estimate |p² - d·q²| < 2·√d + 1 for convergents of √d is stated
/// at the end: its algebraic core (adjacent determinants, denominator growth)
/// is in continued_fraction_convergents.ac and continued_fraction_approx.ac,
/// but the real-analysis statement that the continued fraction of √d
/// converges to √d is not yet formalized, so the general bound remains a
/// statement (as in pell.ac). For d = 2 the exact value |p_n² - 2·q_n²| = 1
/// is proved here, which is the sharpest possible instance of the bound.
from int import Int, abs, add_from_nat, mul_from_nat, neg_sub, sub_self,
    mul_sub_distrib_right, mul_distrib_left, mul_assoc, mul_comm, add_comm,
    add_assoc, add_zero_right, neg_distrib, neg_neg, mul_one_left, add_comm_4
from nat import Nat
from algebra.ring.ring import alternating_sign, alternating_sign_zero,
    alternating_sign_suc
from number_theory.continued_fraction_convergents import continued_fraction_convergent_numerator,
    continued_fraction_convergent_denominator, continued_fraction_alternating_sign_double_suc
from number_theory.continued_fraction_approx import continued_fraction_convergent_numerator_two_suc,
    continued_fraction_convergent_denominator_two_suc, continued_fraction_alternating_sign_one_or_neg_one,
    continued_fraction_alternating_sign_abs_one
from number_theory.pell import sqrt_two_continued_fraction_coefficients, is_pell_solution,
    pell_norm, sqrt_two_convergent_numerator_zero, sqrt_two_convergent_denominator_zero,
    sqrt_two_convergent_numerator_one, sqrt_two_convergent_denominator_one,
    pell_two_fundamental_solution_from_convergents, pell_rearrange_sub_sum,
    pell_rearrange_flatten

numerals Int
numerals Nat

// ============================================================================
// Section 1: the composition step of the convergents of √2
// ============================================================================

/// A numerator two steps ahead satisfies the recurrence p_{k+2} = 2·p_{k+1} + p_k
/// of the continued fraction of √2, with the doubled factor commuted.
theorem cf_pell_compose_step_numerator_comm(x: Nat, y: Nat) {
    x * Nat.2 + y = Nat.2 * x + y
}

/// Expanding the doubled composition step of the fundamental unit:
/// 2·(x + 2·y) + x = (x + 2·y) + (x + 2·y) + x.
theorem cf_pell_compose_step_numerator_expand(x: Nat, y: Nat) {
    Nat.2 * (x + Nat.2 * y) + x = (x + Nat.2 * y) + (x + Nat.2 * y) + x
}

/// Reassociating the expanded composition step.
theorem cf_pell_compose_step_numerator_assoc(x: Nat, y: Nat) {
    (x + Nat.2 * y) + (x + Nat.2 * y) + x =
        (x + Nat.2 * y) + ((x + Nat.2 * y) + x)
}

/// The doubled composition step of the fundamental unit:
/// (x + 2·y) + x = 2·(x + y).
theorem cf_pell_compose_step_numerator_inner(x: Nat, y: Nat) {
    (x + Nat.2 * y) + x = Nat.2 * (x + y)
} by {
    (x + Nat.2 * y) + x = x + Nat.2 * y + x
    x + Nat.2 * y + x = x + x + Nat.2 * y
    x + x + Nat.2 * y = Nat.2 * x + Nat.2 * y
    Nat.2 * x + Nat.2 * y = Nat.2 * (x + y)
    (x + Nat.2 * y) + x = Nat.2 * (x + y)
}

/// The doubled composition step of the fundamental unit:
/// (x + 2·y) + ((x + 2·y) + x) = (x + 2·y) + 2·(x + y).
theorem cf_pell_compose_step_numerator_bridge(x: Nat, y: Nat) {
    (x + Nat.2 * y) + ((x + Nat.2 * y) + x) =
        (x + Nat.2 * y) + Nat.2 * (x + y)
} by {
    cf_pell_compose_step_numerator_inner(x, y)
    (x + Nat.2 * y) + x = Nat.2 * (x + y)
    (x + Nat.2 * y) + ((x + Nat.2 * y) + x) =
        (x + Nat.2 * y) + Nat.2 * (x + y)
}

/// Expanding the doubled denominator composition step:
/// 2·(x + y) + y = (x + y) + (x + y) + y.
theorem cf_pell_compose_step_denominator_expand(x: Nat, y: Nat) {
    Nat.2 * (x + y) + y = (x + y) + (x + y) + y
}

/// Reassociating the expanded denominator composition step.
theorem cf_pell_compose_step_denominator_assoc(x: Nat, y: Nat) {
    (x + y) + (x + y) + y = (x + y) + ((x + y) + y)
}

/// The doubled denominator composition step:
/// (x + y) + ((x + y) + y) = (x + y) + (x + 2·y).
theorem cf_pell_compose_step_denominator_bridge(x: Nat, y: Nat) {
    (x + y) + ((x + y) + y) = (x + y) + (x + Nat.2 * y)
}

/// The zeroth convergent of √2 is obtained from the previous one by the
/// composition step of the fundamental unit: p_1 = p_0 + 2·q_0 and
/// q_1 = p_0 + q_0.
theorem cf_pell_convergent_compose_step_zero {
    continued_fraction_convergent_numerator(
            sqrt_two_continued_fraction_coefficients, Nat.1) =
        continued_fraction_convergent_numerator(
            sqrt_two_continued_fraction_coefficients, Nat.0) +
            Nat.2 * continued_fraction_convergent_denominator(
                sqrt_two_continued_fraction_coefficients, Nat.0) and
    continued_fraction_convergent_denominator(
            sqrt_two_continued_fraction_coefficients, Nat.1) =
        continued_fraction_convergent_numerator(
            sqrt_two_continued_fraction_coefficients, Nat.0) +
            continued_fraction_convergent_denominator(
                sqrt_two_continued_fraction_coefficients, Nat.0)
} by {
    sqrt_two_convergent_numerator_zero
    sqrt_two_convergent_denominator_zero
    sqrt_two_convergent_numerator_one
    sqrt_two_convergent_denominator_one
    continued_fraction_convergent_numerator(
        sqrt_two_continued_fraction_coefficients, Nat.0) = Nat.1
    continued_fraction_convergent_denominator(
        sqrt_two_continued_fraction_coefficients, Nat.0) = Nat.1
    continued_fraction_convergent_numerator(
        sqrt_two_continued_fraction_coefficients, Nat.1) = Nat.3
    continued_fraction_convergent_denominator(
        sqrt_two_continued_fraction_coefficients, Nat.1) = Nat.2
    Nat.1 + Nat.2 * Nat.1 = Nat.3
    Nat.1 + Nat.1 = Nat.2
    continued_fraction_convergent_numerator(
            sqrt_two_continued_fraction_coefficients, Nat.1) =
        continued_fraction_convergent_numerator(
            sqrt_two_continued_fraction_coefficients, Nat.0) +
            Nat.2 * continued_fraction_convergent_denominator(
                sqrt_two_continued_fraction_coefficients, Nat.0)
    continued_fraction_convergent_denominator(
            sqrt_two_continued_fraction_coefficients, Nat.1) =
        continued_fraction_convergent_numerator(
            sqrt_two_continued_fraction_coefficients, Nat.0) +
            continued_fraction_convergent_denominator(
                sqrt_two_continued_fraction_coefficients, Nat.0)
}

/// Each convergent of √2 is obtained from the previous one by the composition
/// step of the fundamental unit 1 + √2: p_{n+1} = p_n + 2·q_n and
/// q_{n+1} = p_n + q_n.
theorem cf_pell_convergent_compose_step(n: Nat) {
    continued_fraction_convergent_numerator(
            sqrt_two_continued_fraction_coefficients, n.suc) =
        continued_fraction_convergent_numerator(
            sqrt_two_continued_fraction_coefficients, n) +
            Nat.2 * continued_fraction_convergent_denominator(
                sqrt_two_continued_fraction_coefficients, n) and
    continued_fraction_convergent_denominator(
            sqrt_two_continued_fraction_coefficients, n.suc) =
        continued_fraction_convergent_numerator(
            sqrt_two_continued_fraction_coefficients, n) +
            continued_fraction_convergent_denominator(
                sqrt_two_continued_fraction_coefficients, n)
} by {
    define step(k: Nat) -> Bool {
        continued_fraction_convergent_numerator(
                sqrt_two_continued_fraction_coefficients, k.suc) =
            continued_fraction_convergent_numerator(
                sqrt_two_continued_fraction_coefficients, k) +
                Nat.2 * continued_fraction_convergent_denominator(
                    sqrt_two_continued_fraction_coefficients, k) and
        continued_fraction_convergent_denominator(
                sqrt_two_continued_fraction_coefficients, k.suc) =
            continued_fraction_convergent_numerator(
                sqrt_two_continued_fraction_coefficients, k) +
                continued_fraction_convergent_denominator(
                    sqrt_two_continued_fraction_coefficients, k)
    }
    cf_pell_convergent_compose_step_zero
    step(Nat.0)
    forall(k: Nat) {
        if step(k) {
            continued_fraction_convergent_numerator(
                    sqrt_two_continued_fraction_coefficients, k.suc) =
                continued_fraction_convergent_numerator(
                    sqrt_two_continued_fraction_coefficients, k) +
                    Nat.2 * continued_fraction_convergent_denominator(
                        sqrt_two_continued_fraction_coefficients, k)
            continued_fraction_convergent_denominator(
                    sqrt_two_continued_fraction_coefficients, k.suc) =
                continued_fraction_convergent_numerator(
                    sqrt_two_continued_fraction_coefficients, k) +
                    continued_fraction_convergent_denominator(
                        sqrt_two_continued_fraction_coefficients, k)
            continued_fraction_convergent_numerator_two_suc(
                sqrt_two_continued_fraction_coefficients, k)
            continued_fraction_convergent_numerator(
                    sqrt_two_continued_fraction_coefficients, k.suc.suc) =
                continued_fraction_convergent_numerator(
                        sqrt_two_continued_fraction_coefficients, k.suc) *
                        sqrt_two_continued_fraction_coefficients(k.suc.suc) +
                    continued_fraction_convergent_numerator(
                        sqrt_two_continued_fraction_coefficients, k)
            sqrt_two_continued_fraction_coefficients(k.suc.suc) = Nat.2
            continued_fraction_convergent_numerator(
                    sqrt_two_continued_fraction_coefficients, k.suc.suc) =
                continued_fraction_convergent_numerator(
                        sqrt_two_continued_fraction_coefficients, k.suc) *
                        Nat.2 +
                    continued_fraction_convergent_numerator(
                        sqrt_two_continued_fraction_coefficients, k)
            cf_pell_compose_step_numerator_comm(
                continued_fraction_convergent_numerator(
                    sqrt_two_continued_fraction_coefficients, k.suc),
                continued_fraction_convergent_numerator(
                    sqrt_two_continued_fraction_coefficients, k))
            continued_fraction_convergent_numerator(
                    sqrt_two_continued_fraction_coefficients, k.suc.suc) =
                Nat.2 * continued_fraction_convergent_numerator(
                        sqrt_two_continued_fraction_coefficients, k.suc) +
                    continued_fraction_convergent_numerator(
                        sqrt_two_continued_fraction_coefficients, k)
            continued_fraction_convergent_numerator(
                    sqrt_two_continued_fraction_coefficients, k.suc.suc) =
                Nat.2 * (continued_fraction_convergent_numerator(
                        sqrt_two_continued_fraction_coefficients, k) +
                    Nat.2 * continued_fraction_convergent_denominator(
                        sqrt_two_continued_fraction_coefficients, k)) +
                    continued_fraction_convergent_numerator(
                        sqrt_two_continued_fraction_coefficients, k)
            cf_pell_compose_step_numerator_expand(
                continued_fraction_convergent_numerator(
                    sqrt_two_continued_fraction_coefficients, k),
                continued_fraction_convergent_denominator(
                    sqrt_two_continued_fraction_coefficients, k))
            continued_fraction_convergent_numerator(
                    sqrt_two_continued_fraction_coefficients, k.suc.suc) =
                (continued_fraction_convergent_numerator(
                        sqrt_two_continued_fraction_coefficients, k) +
                    Nat.2 * continued_fraction_convergent_denominator(
                        sqrt_two_continued_fraction_coefficients, k)) +
                    (continued_fraction_convergent_numerator(
                        sqrt_two_continued_fraction_coefficients, k) +
                    Nat.2 * continued_fraction_convergent_denominator(
                        sqrt_two_continued_fraction_coefficients, k)) +
                    continued_fraction_convergent_numerator(
                        sqrt_two_continued_fraction_coefficients, k)
            cf_pell_compose_step_numerator_assoc(
                continued_fraction_convergent_numerator(
                    sqrt_two_continued_fraction_coefficients, k),
                continued_fraction_convergent_denominator(
                    sqrt_two_continued_fraction_coefficients, k))
            continued_fraction_convergent_numerator(
                    sqrt_two_continued_fraction_coefficients, k.suc.suc) =
                (continued_fraction_convergent_numerator(
                        sqrt_two_continued_fraction_coefficients, k) +
                    Nat.2 * continued_fraction_convergent_denominator(
                        sqrt_two_continued_fraction_coefficients, k)) +
                    ((continued_fraction_convergent_numerator(
                            sqrt_two_continued_fraction_coefficients, k) +
                        Nat.2 * continued_fraction_convergent_denominator(
                            sqrt_two_continued_fraction_coefficients, k)) +
                    continued_fraction_convergent_numerator(
                        sqrt_two_continued_fraction_coefficients, k))
            cf_pell_compose_step_numerator_bridge(
                continued_fraction_convergent_numerator(
                    sqrt_two_continued_fraction_coefficients, k),
                continued_fraction_convergent_denominator(
                    sqrt_two_continued_fraction_coefficients, k))
            continued_fraction_convergent_numerator(
                    sqrt_two_continued_fraction_coefficients, k.suc.suc) =
                (continued_fraction_convergent_numerator(
                        sqrt_two_continued_fraction_coefficients, k) +
                    Nat.2 * continued_fraction_convergent_denominator(
                        sqrt_two_continued_fraction_coefficients, k)) +
                    Nat.2 * (continued_fraction_convergent_numerator(
                            sqrt_two_continued_fraction_coefficients, k) +
                        continued_fraction_convergent_denominator(
                            sqrt_two_continued_fraction_coefficients, k))
            continued_fraction_convergent_numerator(
                    sqrt_two_continued_fraction_coefficients, k.suc.suc) =
                continued_fraction_convergent_numerator(
                        sqrt_two_continued_fraction_coefficients, k.suc) +
                    Nat.2 * continued_fraction_convergent_denominator(
                        sqrt_two_continued_fraction_coefficients, k.suc)
            continued_fraction_convergent_denominator_two_suc(
                sqrt_two_continued_fraction_coefficients, k)
            continued_fraction_convergent_denominator(
                    sqrt_two_continued_fraction_coefficients, k.suc.suc) =
                continued_fraction_convergent_denominator(
                        sqrt_two_continued_fraction_coefficients, k.suc) *
                        sqrt_two_continued_fraction_coefficients(k.suc.suc) +
                    continued_fraction_convergent_denominator(
                        sqrt_two_continued_fraction_coefficients, k)
            continued_fraction_convergent_denominator(
                    sqrt_two_continued_fraction_coefficients, k.suc.suc) =
                continued_fraction_convergent_denominator(
                        sqrt_two_continued_fraction_coefficients, k.suc) *
                        Nat.2 +
                    continued_fraction_convergent_denominator(
                        sqrt_two_continued_fraction_coefficients, k)
            cf_pell_compose_step_numerator_comm(
                continued_fraction_convergent_denominator(
                    sqrt_two_continued_fraction_coefficients, k.suc),
                continued_fraction_convergent_denominator(
                    sqrt_two_continued_fraction_coefficients, k))
            continued_fraction_convergent_denominator(
                    sqrt_two_continued_fraction_coefficients, k.suc.suc) =
                Nat.2 * continued_fraction_convergent_denominator(
                        sqrt_two_continued_fraction_coefficients, k.suc) +
                    continued_fraction_convergent_denominator(
                        sqrt_two_continued_fraction_coefficients, k)
            continued_fraction_convergent_denominator(
                    sqrt_two_continued_fraction_coefficients, k.suc.suc) =
                Nat.2 * (continued_fraction_convergent_numerator(
                        sqrt_two_continued_fraction_coefficients, k) +
                    continued_fraction_convergent_denominator(
                        sqrt_two_continued_fraction_coefficients, k)) +
                    continued_fraction_convergent_denominator(
                        sqrt_two_continued_fraction_coefficients, k)
            cf_pell_compose_step_denominator_expand(
                continued_fraction_convergent_numerator(
                    sqrt_two_continued_fraction_coefficients, k),
                continued_fraction_convergent_denominator(
                    sqrt_two_continued_fraction_coefficients, k))
            continued_fraction_convergent_denominator(
                    sqrt_two_continued_fraction_coefficients, k.suc.suc) =
                (continued_fraction_convergent_numerator(
                        sqrt_two_continued_fraction_coefficients, k) +
                    continued_fraction_convergent_denominator(
                        sqrt_two_continued_fraction_coefficients, k)) +
                    (continued_fraction_convergent_numerator(
                        sqrt_two_continued_fraction_coefficients, k) +
                    continued_fraction_convergent_denominator(
                        sqrt_two_continued_fraction_coefficients, k)) +
                    continued_fraction_convergent_denominator(
                        sqrt_two_continued_fraction_coefficients, k)
            cf_pell_compose_step_denominator_assoc(
                continued_fraction_convergent_numerator(
                    sqrt_two_continued_fraction_coefficients, k),
                continued_fraction_convergent_denominator(
                    sqrt_two_continued_fraction_coefficients, k))
            continued_fraction_convergent_denominator(
                    sqrt_two_continued_fraction_coefficients, k.suc.suc) =
                (continued_fraction_convergent_numerator(
                        sqrt_two_continued_fraction_coefficients, k) +
                    continued_fraction_convergent_denominator(
                        sqrt_two_continued_fraction_coefficients, k)) +
                    ((continued_fraction_convergent_numerator(
                            sqrt_two_continued_fraction_coefficients, k) +
                        continued_fraction_convergent_denominator(
                            sqrt_two_continued_fraction_coefficients, k)) +
                    continued_fraction_convergent_denominator(
                        sqrt_two_continued_fraction_coefficients, k))
            cf_pell_compose_step_denominator_bridge(
                continued_fraction_convergent_numerator(
                    sqrt_two_continued_fraction_coefficients, k),
                continued_fraction_convergent_denominator(
                    sqrt_two_continued_fraction_coefficients, k))
            continued_fraction_convergent_denominator(
                    sqrt_two_continued_fraction_coefficients, k.suc.suc) =
                (continued_fraction_convergent_numerator(
                        sqrt_two_continued_fraction_coefficients, k) +
                    continued_fraction_convergent_denominator(
                        sqrt_two_continued_fraction_coefficients, k)) +
                    (continued_fraction_convergent_numerator(
                            sqrt_two_continued_fraction_coefficients, k) +
                        Nat.2 * continued_fraction_convergent_denominator(
                            sqrt_two_continued_fraction_coefficients, k))
            continued_fraction_convergent_denominator(
                    sqrt_two_continued_fraction_coefficients, k.suc.suc) =
                continued_fraction_convergent_numerator(
                        sqrt_two_continued_fraction_coefficients, k.suc) +
                    continued_fraction_convergent_denominator(
                        sqrt_two_continued_fraction_coefficients, k.suc)
            step(k.suc)
        }
        step(k) implies step(k.suc)
    }
    step(Nat.0) and forall(k: Nat) { step(k) implies step(k.suc) }
    Nat.induction(step)
    forall(k: Nat) { step(k) }
    step(n)
}

// ============================================================================
// Section 2: the norm of a convergent of √2
// ============================================================================

/// Subtraction distributes over sums pairwise:
/// (a + b) - (c + d) = (a - c) + (b - d).
theorem cf_pell_sub_pair_identity(a: Int, b: Int, c: Int, d: Int) {
    (a + b) - (c + d) = (a - c) + (b - d)
} by {
    (a + b) - (c + d) = (a + b) + -(c + d)
    neg_distrib(c, d)
    -(c + d) = -c + -d
    (a + b) - (c + d) = (a + b) + (-c + -d)
    (a + b) + (-c + -d) = a + b + (-c + -d)
    a + b + (-c + -d) = a + b - c - d
    (a + b) - (c + d) = a + b - c - d
    (a - c) + (b - d) = (a + -c) + (b + -d)
    (a + -c) + (b + -d) = a + -c + b + -d
    a + -c + b + -d = a + b + -c + -d
    a + b + -c + -d = a + b - c - d
    (a - c) + (b - d) = a + b - c - d
    (a + b) - (c + d) = (a - c) + (b - d)
}

/// The square coefficient regroup: x² - 2·x² = -x².
theorem cf_pell_norm_x2_regroup(x: Int, y: Int) {
    x * x - Int.2 * (x * x) = -x * x
} by {
    x * x - Int.2 * (x * x) = (Int.1 - Int.2) * (x * x)
    Int.1 - Int.2 = -Int.1
    (Int.1 - Int.2) * (x * x) = -Int.1 * (x * x)
    -Int.1 * (x * x) = -(x * x)
    x * x - Int.2 * (x * x) = -x * x
}

/// The square coefficient regroup: 4·y² - 2·y² = 2·y².
theorem cf_pell_norm_y2_regroup(x: Int, y: Int) {
    Int.4 * (y * y) - Int.2 * (y * y) = Int.2 * (y * y)
} by {
    Int.4 * (y * y) - Int.2 * (y * y) = (Int.4 - Int.2) * (y * y)
    add_from_nat(Nat.2, Nat.2)
    Int.from_nat(Nat.2) + Int.from_nat(Nat.2) = Int.from_nat(Nat.4)
    Int.from_nat(Nat.2) = Int.2
    Int.from_nat(Nat.4) = Int.4
    Int.2 + Int.2 = Int.4
    Int.4 = Int.2 + Int.2
    (Int.2 + Int.2) - Int.2 = Int.2
    Int.4 - Int.2 = Int.2
    (Int.4 - Int.2) * (y * y) = Int.2 * (y * y)
    Int.4 * (y * y) - Int.2 * (y * y) = Int.2 * (y * y)
}

/// Composing a pair with the fundamental unit 1 + √2 flips the sign of the
/// Pell norm: (x + 2·y)² - 2·(x + y)² = -(x² - 2·y²).
theorem cf_pell_norm_flip_compose_one(x: Int, y: Int) {
    pell_norm(Int.2, x + Int.2 * y, x + y) = -pell_norm(Int.2, x, y)
} by {
    pell_norm(Int.2, x + Int.2 * y, x + y) =
        (x + Int.2 * y) * (x + Int.2 * y) - Int.2 * ((x + y) * (x + y))
    (x + Int.2 * y) * (x + Int.2 * y) =
        (x + Int.2 * y) * x + (x + Int.2 * y) * (Int.2 * y)
    (x + Int.2 * y) * x = x * x + (Int.2 * y) * x
    (Int.2 * y) * x = Int.2 * (x * y)
    (x + Int.2 * y) * (Int.2 * y) = x * (Int.2 * y) + (Int.2 * y) * (Int.2 * y)
    x * (Int.2 * y) = Int.2 * (x * y)
    (Int.2 * y) * (Int.2 * y) = Int.2 * (y * (Int.2 * y))
    y * (Int.2 * y) = Int.2 * (y * y)
    Int.2 * (Int.2 * (y * y)) = (Int.2 * Int.2) * (y * y)
    mul_from_nat(Nat.2, Nat.2)
    Int.from_nat(Nat.2) * Int.from_nat(Nat.2) = Int.from_nat(Nat.4)
    Int.from_nat(Nat.2) = Int.2
    Int.from_nat(Nat.4) = Int.4
    Int.2 * Int.2 = Int.4
    (Int.2 * Int.2) * (y * y) = Int.4 * (y * y)
    Int.2 * (Int.2 * (y * y)) = Int.4 * (y * y)
    (x + Int.2 * y) * (x + Int.2 * y) =
        x * x + Int.2 * (x * y) + Int.2 * (x * y) + Int.4 * (y * y)
    Int.2 * (x * y) + Int.2 * (x * y) = Int.2 * (Int.2 * (x * y))
    Int.2 * (Int.2 * (x * y)) = Int.4 * (x * y)
    (x + Int.2 * y) * (x + Int.2 * y) =
        x * x + Int.4 * (x * y) + Int.4 * (y * y)
    (x + y) * (x + y) = (x + y) * x + (x + y) * y
    (x + y) * x = x * x + y * x
    y * x = x * y
    (x + y) * y = x * y + y * y
    (x + y) * (x + y) = x * x + x * y + x * y + y * y
    x * y + x * y = Int.2 * (x * y)
    (x + y) * (x + y) = x * x + Int.2 * (x * y) + y * y
    pell_norm(Int.2, x + Int.2 * y, x + y) =
        (x * x + Int.4 * (x * y) + Int.4 * (y * y)) -
            Int.2 * (x * x + Int.2 * (x * y) + y * y)
    Int.2 * (x * x + Int.2 * (x * y) + y * y) =
        Int.2 * (x * x) + Int.2 * (Int.2 * (x * y)) + Int.2 * (y * y)
    Int.2 * (Int.2 * (x * y)) = Int.4 * (x * y)
    Int.2 * (x * x + Int.2 * (x * y) + y * y) =
        Int.2 * (x * x) + Int.4 * (x * y) + Int.2 * (y * y)
    pell_norm(Int.2, x + Int.2 * y, x + y) =
        (x * x + Int.4 * (x * y) + Int.4 * (y * y)) -
            (Int.2 * (x * x) + Int.4 * (x * y) + Int.2 * (y * y))
    x * x + Int.4 * (x * y) + Int.4 * (y * y) =
        x * x + Int.4 * (y * y) + Int.4 * (x * y)
    Int.2 * (x * x) + Int.4 * (x * y) + Int.2 * (y * y) =
        Int.2 * (x * x) + Int.2 * (y * y) + Int.4 * (x * y)
    pell_norm(Int.2, x + Int.2 * y, x + y) =
        (x * x + Int.4 * (y * y) + Int.4 * (x * y)) -
            (Int.2 * (x * x) + Int.2 * (y * y) + Int.4 * (x * y))
    pell_norm(Int.2, x + Int.2 * y, x + y) =
        (x * x + Int.4 * (y * y)) - (Int.2 * (x * x) + Int.2 * (y * y))
    pell_rearrange_sub_sum(x * x + Int.4 * (y * y), Int.2 * (x * x), Int.2 * (y * y))
    (x * x + Int.4 * (y * y)) - (Int.2 * (x * x) + Int.2 * (y * y)) =
        (x * x + Int.4 * (y * y)) - Int.2 * (x * x) - Int.2 * (y * y)
    pell_rearrange_flatten(x * x, Int.4 * (y * y), Int.2 * (x * x), Int.2 * (y * y))
    (x * x + Int.4 * (y * y)) - Int.2 * (x * x) - Int.2 * (y * y) =
        x * x + Int.4 * (y * y) - Int.2 * (x * x) - Int.2 * (y * y)
    pell_norm(Int.2, x + Int.2 * y, x + y) =
        x * x + Int.4 * (y * y) - Int.2 * (x * x) - Int.2 * (y * y)
    cf_pell_sub_pair_identity(x * x, Int.4 * (y * y), Int.2 * (x * x), Int.2 * (y * y))
    x * x + Int.4 * (y * y) - Int.2 * (x * x) - Int.2 * (y * y) =
        (x * x - Int.2 * (x * x)) + (Int.4 * (y * y) - Int.2 * (y * y))
    cf_pell_norm_x2_regroup(x, y)
    cf_pell_norm_y2_regroup(x, y)
    x * x - Int.2 * (x * x) = -x * x
    Int.4 * (y * y) - Int.2 * (y * y) = Int.2 * (y * y)
    (x * x - Int.2 * (x * x)) + (Int.4 * (y * y) - Int.2 * (y * y)) =
        -x * x + Int.2 * (y * y)
    pell_norm(Int.2, x + Int.2 * y, x + y) = -x * x + Int.2 * (y * y)
    pell_norm(Int.2, x, y) = x * x - Int.2 * (y * y)
    -pell_norm(Int.2, x, y) = -(x * x - Int.2 * (y * y))
    -(x * x - Int.2 * (y * y)) = -x * x + Int.2 * (y * y)
    -pell_norm(Int.2, x, y) = -x * x + Int.2 * (y * y)
    pell_norm(Int.2, x + Int.2 * y, x + y) = -pell_norm(Int.2, x, y)
}

/// The norm of the n-th convergent of √2 is the alternating sign at the next
/// index: p_n² - 2·q_n² = (-1)^(n+1).
theorem cf_pell_convergent_norm_alternating(n: Nat) {
    pell_norm(Int.2,
        Int.from_nat(continued_fraction_convergent_numerator(
            sqrt_two_continued_fraction_coefficients, n)),
        Int.from_nat(continued_fraction_convergent_denominator(
            sqrt_two_continued_fraction_coefficients, n))) =
        alternating_sign[Int](n.suc)
} by {
    define p(k: Nat) -> Bool {
        pell_norm(Int.2,
            Int.from_nat(continued_fraction_convergent_numerator(
                sqrt_two_continued_fraction_coefficients, k)),
            Int.from_nat(continued_fraction_convergent_denominator(
                sqrt_two_continued_fraction_coefficients, k))) =
            alternating_sign[Int](k.suc)
    }
    sqrt_two_convergent_numerator_zero
    sqrt_two_convergent_denominator_zero
    continued_fraction_convergent_numerator(
        sqrt_two_continued_fraction_coefficients, Nat.0) = Nat.1
    continued_fraction_convergent_denominator(
        sqrt_two_continued_fraction_coefficients, Nat.0) = Nat.1
    Int.from_nat(Nat.1) = Int.1
    pell_norm(Int.2, Int.1, Int.1) = Int.1 * Int.1 - Int.2 * (Int.1 * Int.1)
    Int.1 * Int.1 = Int.1
    pell_norm(Int.2, Int.1, Int.1) = Int.1 - Int.2
    Int.1 - Int.2 = -Int.1
    pell_norm(Int.2, Int.1, Int.1) = -Int.1
    alternating_sign_zero[Int]
    alternating_sign[Int](Nat.0) = Int.1
    alternating_sign_suc[Int](Nat.0)
    alternating_sign[Int](Nat.1) = -alternating_sign[Int](Nat.0)
    alternating_sign[Int](Nat.1) = -Int.1
    pell_norm(Int.2,
        Int.from_nat(continued_fraction_convergent_numerator(
            sqrt_two_continued_fraction_coefficients, Nat.0)),
        Int.from_nat(continued_fraction_convergent_denominator(
            sqrt_two_continued_fraction_coefficients, Nat.0))) =
        alternating_sign[Int](Nat.1)
    p(Nat.0)
    forall(k: Nat) {
        if p(k) {
            p(k)
            pell_norm(Int.2,
                Int.from_nat(continued_fraction_convergent_numerator(
                    sqrt_two_continued_fraction_coefficients, k)),
                Int.from_nat(continued_fraction_convergent_denominator(
                    sqrt_two_continued_fraction_coefficients, k))) =
                alternating_sign[Int](k.suc)
            cf_pell_convergent_compose_step(k)
            continued_fraction_convergent_numerator(
                    sqrt_two_continued_fraction_coefficients, k.suc) =
                continued_fraction_convergent_numerator(
                    sqrt_two_continued_fraction_coefficients, k) +
                    Nat.2 * continued_fraction_convergent_denominator(
                        sqrt_two_continued_fraction_coefficients, k)
            continued_fraction_convergent_denominator(
                    sqrt_two_continued_fraction_coefficients, k.suc) =
                continued_fraction_convergent_numerator(
                    sqrt_two_continued_fraction_coefficients, k) +
                    continued_fraction_convergent_denominator(
                        sqrt_two_continued_fraction_coefficients, k)
            pell_norm(Int.2,
                Int.from_nat(continued_fraction_convergent_numerator(
                    sqrt_two_continued_fraction_coefficients, k.suc)),
                Int.from_nat(continued_fraction_convergent_denominator(
                    sqrt_two_continued_fraction_coefficients, k.suc))) =
                pell_norm(Int.2,
                    Int.from_nat(continued_fraction_convergent_numerator(
                            sqrt_two_continued_fraction_coefficients, k) +
                        Nat.2 * continued_fraction_convergent_denominator(
                            sqrt_two_continued_fraction_coefficients, k)),
                    Int.from_nat(continued_fraction_convergent_numerator(
                            sqrt_two_continued_fraction_coefficients, k) +
                        continued_fraction_convergent_denominator(
                            sqrt_two_continued_fraction_coefficients, k)))
            add_from_nat(
                continued_fraction_convergent_numerator(
                    sqrt_two_continued_fraction_coefficients, k),
                Nat.2 * continued_fraction_convergent_denominator(
                    sqrt_two_continued_fraction_coefficients, k))
            mul_from_nat(Nat.2,
                continued_fraction_convergent_denominator(
                    sqrt_two_continued_fraction_coefficients, k))
            add_from_nat(
                continued_fraction_convergent_numerator(
                    sqrt_two_continued_fraction_coefficients, k),
                continued_fraction_convergent_denominator(
                    sqrt_two_continued_fraction_coefficients, k))
            Int.from_nat(continued_fraction_convergent_numerator(
                        sqrt_two_continued_fraction_coefficients, k) +
                    Nat.2 * continued_fraction_convergent_denominator(
                        sqrt_two_continued_fraction_coefficients, k)) =
                Int.from_nat(continued_fraction_convergent_numerator(
                        sqrt_two_continued_fraction_coefficients, k)) +
                    Int.2 * Int.from_nat(continued_fraction_convergent_denominator(
                        sqrt_two_continued_fraction_coefficients, k))
            Int.from_nat(continued_fraction_convergent_numerator(
                        sqrt_two_continued_fraction_coefficients, k) +
                    continued_fraction_convergent_denominator(
                        sqrt_two_continued_fraction_coefficients, k)) =
                Int.from_nat(continued_fraction_convergent_numerator(
                        sqrt_two_continued_fraction_coefficients, k)) +
                    Int.from_nat(continued_fraction_convergent_denominator(
                        sqrt_two_continued_fraction_coefficients, k))
            pell_norm(Int.2,
                Int.from_nat(continued_fraction_convergent_numerator(
                    sqrt_two_continued_fraction_coefficients, k.suc)),
                Int.from_nat(continued_fraction_convergent_denominator(
                    sqrt_two_continued_fraction_coefficients, k.suc))) =
                pell_norm(Int.2,
                    Int.from_nat(continued_fraction_convergent_numerator(
                            sqrt_two_continued_fraction_coefficients, k)) +
                        Int.2 * Int.from_nat(continued_fraction_convergent_denominator(
                            sqrt_two_continued_fraction_coefficients, k)),
                    Int.from_nat(continued_fraction_convergent_numerator(
                            sqrt_two_continued_fraction_coefficients, k)) +
                        Int.from_nat(continued_fraction_convergent_denominator(
                            sqrt_two_continued_fraction_coefficients, k)))
            cf_pell_norm_flip_compose_one(
                Int.from_nat(continued_fraction_convergent_numerator(
                    sqrt_two_continued_fraction_coefficients, k)),
                Int.from_nat(continued_fraction_convergent_denominator(
                    sqrt_two_continued_fraction_coefficients, k)))
            pell_norm(Int.2,
                Int.from_nat(continued_fraction_convergent_numerator(
                        sqrt_two_continued_fraction_coefficients, k)) +
                    Int.2 * Int.from_nat(continued_fraction_convergent_denominator(
                        sqrt_two_continued_fraction_coefficients, k)),
                Int.from_nat(continued_fraction_convergent_numerator(
                        sqrt_two_continued_fraction_coefficients, k)) +
                    Int.from_nat(continued_fraction_convergent_denominator(
                        sqrt_two_continued_fraction_coefficients, k))) =
                -pell_norm(Int.2,
                    Int.from_nat(continued_fraction_convergent_numerator(
                        sqrt_two_continued_fraction_coefficients, k)),
                    Int.from_nat(continued_fraction_convergent_denominator(
                        sqrt_two_continued_fraction_coefficients, k)))
            pell_norm(Int.2,
                Int.from_nat(continued_fraction_convergent_numerator(
                    sqrt_two_continued_fraction_coefficients, k.suc)),
                Int.from_nat(continued_fraction_convergent_denominator(
                    sqrt_two_continued_fraction_coefficients, k.suc))) =
                -pell_norm(Int.2,
                    Int.from_nat(continued_fraction_convergent_numerator(
                        sqrt_two_continued_fraction_coefficients, k)),
                    Int.from_nat(continued_fraction_convergent_denominator(
                        sqrt_two_continued_fraction_coefficients, k)))
            pell_norm(Int.2,
                Int.from_nat(continued_fraction_convergent_numerator(
                    sqrt_two_continued_fraction_coefficients, k.suc)),
                Int.from_nat(continued_fraction_convergent_denominator(
                    sqrt_two_continued_fraction_coefficients, k.suc))) =
                -alternating_sign[Int](k.suc)
            alternating_sign_suc[Int](k.suc)
            alternating_sign[Int](k.suc.suc) = -alternating_sign[Int](k.suc)
            pell_norm(Int.2,
                Int.from_nat(continued_fraction_convergent_numerator(
                    sqrt_two_continued_fraction_coefficients, k.suc)),
                Int.from_nat(continued_fraction_convergent_denominator(
                    sqrt_two_continued_fraction_coefficients, k.suc))) =
                alternating_sign[Int](k.suc.suc)
            p(k.suc)
        }
        p(k) implies p(k.suc)
    }
    p(Nat.0) and forall(k: Nat) { p(k) implies p(k.suc) }
    Nat.induction(p)
    forall(k: Nat) { p(k) }
    p(n)
}

// ============================================================================
// Section 3: the classical identity p_n² - 2·q_n² = ±1
// ============================================================================

/// Every convergent of √2 has norm one or minus one:
/// p_n² - 2·q_n² = ±1.
theorem cf_pell_convergent_norm_one_or_neg_one(n: Nat) {
    pell_norm(Int.2,
        Int.from_nat(continued_fraction_convergent_numerator(
            sqrt_two_continued_fraction_coefficients, n)),
        Int.from_nat(continued_fraction_convergent_denominator(
            sqrt_two_continued_fraction_coefficients, n))) = Int.1 or
    pell_norm(Int.2,
        Int.from_nat(continued_fraction_convergent_numerator(
            sqrt_two_continued_fraction_coefficients, n)),
        Int.from_nat(continued_fraction_convergent_denominator(
            sqrt_two_continued_fraction_coefficients, n))) = -Int.1
} by {
    cf_pell_convergent_norm_alternating(n)
    continued_fraction_alternating_sign_one_or_neg_one(n.suc)
    pell_norm(Int.2,
        Int.from_nat(continued_fraction_convergent_numerator(
            sqrt_two_continued_fraction_coefficients, n)),
        Int.from_nat(continued_fraction_convergent_denominator(
            sqrt_two_continued_fraction_coefficients, n))) =
        alternating_sign[Int](n.suc)
    alternating_sign[Int](n.suc) = Int.1 or alternating_sign[Int](n.suc) = -Int.1
    if alternating_sign[Int](n.suc) = Int.1 {
        pell_norm(Int.2,
            Int.from_nat(continued_fraction_convergent_numerator(
                sqrt_two_continued_fraction_coefficients, n)),
            Int.from_nat(continued_fraction_convergent_denominator(
                sqrt_two_continued_fraction_coefficients, n))) = Int.1
        pell_norm(Int.2,
            Int.from_nat(continued_fraction_convergent_numerator(
                sqrt_two_continued_fraction_coefficients, n)),
            Int.from_nat(continued_fraction_convergent_denominator(
                sqrt_two_continued_fraction_coefficients, n))) = Int.1 or
            pell_norm(Int.2,
                Int.from_nat(continued_fraction_convergent_numerator(
                    sqrt_two_continued_fraction_coefficients, n)),
                Int.from_nat(continued_fraction_convergent_denominator(
                    sqrt_two_continued_fraction_coefficients, n))) = -Int.1
    }
    if alternating_sign[Int](n.suc) = -Int.1 {
        pell_norm(Int.2,
            Int.from_nat(continued_fraction_convergent_numerator(
                sqrt_two_continued_fraction_coefficients, n)),
            Int.from_nat(continued_fraction_convergent_denominator(
                sqrt_two_continued_fraction_coefficients, n))) = -Int.1
        pell_norm(Int.2,
            Int.from_nat(continued_fraction_convergent_numerator(
                sqrt_two_continued_fraction_coefficients, n)),
            Int.from_nat(continued_fraction_convergent_denominator(
                sqrt_two_continued_fraction_coefficients, n))) = Int.1 or
            pell_norm(Int.2,
                Int.from_nat(continued_fraction_convergent_numerator(
                    sqrt_two_continued_fraction_coefficients, n)),
                Int.from_nat(continued_fraction_convergent_denominator(
                    sqrt_two_continued_fraction_coefficients, n))) = -Int.1
    }
    pell_norm(Int.2,
        Int.from_nat(continued_fraction_convergent_numerator(
            sqrt_two_continued_fraction_coefficients, n)),
        Int.from_nat(continued_fraction_convergent_denominator(
            sqrt_two_continued_fraction_coefficients, n))) = Int.1 or
        pell_norm(Int.2,
            Int.from_nat(continued_fraction_convergent_numerator(
                sqrt_two_continued_fraction_coefficients, n)),
            Int.from_nat(continued_fraction_convergent_denominator(
                sqrt_two_continued_fraction_coefficients, n))) = -Int.1
}

/// The absolute value of the norm of a convergent of √2 is one:
/// |p_n² - 2·q_n²| = 1.
theorem cf_pell_convergent_norm_abs_one(n: Nat) {
    abs(pell_norm(Int.2,
        Int.from_nat(continued_fraction_convergent_numerator(
            sqrt_two_continued_fraction_coefficients, n)),
        Int.from_nat(continued_fraction_convergent_denominator(
            sqrt_two_continued_fraction_coefficients, n)))) = Nat.1
} by {
    cf_pell_convergent_norm_alternating(n)
    continued_fraction_alternating_sign_abs_one(n.suc)
    pell_norm(Int.2,
        Int.from_nat(continued_fraction_convergent_numerator(
            sqrt_two_continued_fraction_coefficients, n)),
        Int.from_nat(continued_fraction_convergent_denominator(
            sqrt_two_continued_fraction_coefficients, n))) =
        alternating_sign[Int](n.suc)
    abs(alternating_sign[Int](n.suc)) = Nat.1
    abs(pell_norm(Int.2,
        Int.from_nat(continued_fraction_convergent_numerator(
            sqrt_two_continued_fraction_coefficients, n)),
        Int.from_nat(continued_fraction_convergent_denominator(
            sqrt_two_continued_fraction_coefficients, n)))) = Nat.1
}

// ============================================================================
// Section 4: convergents of √2 and Pell's equation
// ============================================================================

/// Every odd-indexed convergent of √2 solves Pell's equation x² - 2·y² = 1.
theorem cf_pell_odd_convergent_pell_solution(k: Nat) {
    is_pell_solution(Int.2,
        Int.from_nat(continued_fraction_convergent_numerator(
            sqrt_two_continued_fraction_coefficients, (Nat.2 * k).suc)),
        Int.from_nat(continued_fraction_convergent_denominator(
            sqrt_two_continued_fraction_coefficients, (Nat.2 * k).suc)))
} by {
    continued_fraction_alternating_sign_double_suc(k)
    alternating_sign[Int]((Nat.2 * k).suc) = -Int.1
    alternating_sign_suc[Int]((Nat.2 * k).suc)
    alternating_sign[Int]((Nat.2 * k).suc.suc) =
        -alternating_sign[Int]((Nat.2 * k).suc)
    alternating_sign[Int]((Nat.2 * k).suc.suc) = -(-Int.1)
    --Int.1 = Int.1
    alternating_sign[Int]((Nat.2 * k).suc.suc) = Int.1
    cf_pell_convergent_norm_alternating((Nat.2 * k).suc)
    pell_norm(Int.2,
        Int.from_nat(continued_fraction_convergent_numerator(
            sqrt_two_continued_fraction_coefficients, (Nat.2 * k).suc)),
        Int.from_nat(continued_fraction_convergent_denominator(
            sqrt_two_continued_fraction_coefficients, (Nat.2 * k).suc))) =
        alternating_sign[Int]((Nat.2 * k).suc.suc)
    pell_norm(Int.2,
        Int.from_nat(continued_fraction_convergent_numerator(
            sqrt_two_continued_fraction_coefficients, (Nat.2 * k).suc)),
        Int.from_nat(continued_fraction_convergent_denominator(
            sqrt_two_continued_fraction_coefficients, (Nat.2 * k).suc))) = Int.1
    pell_norm(Int.2,
        Int.from_nat(continued_fraction_convergent_numerator(
            sqrt_two_continued_fraction_coefficients, (Nat.2 * k).suc)),
        Int.from_nat(continued_fraction_convergent_denominator(
            sqrt_two_continued_fraction_coefficients, (Nat.2 * k).suc))) =
        Int.from_nat(continued_fraction_convergent_numerator(
            sqrt_two_continued_fraction_coefficients, (Nat.2 * k).suc)) *
            Int.from_nat(continued_fraction_convergent_numerator(
                sqrt_two_continued_fraction_coefficients, (Nat.2 * k).suc)) -
        Int.2 * (Int.from_nat(continued_fraction_convergent_denominator(
                sqrt_two_continued_fraction_coefficients, (Nat.2 * k).suc)) *
            Int.from_nat(continued_fraction_convergent_denominator(
                sqrt_two_continued_fraction_coefficients, (Nat.2 * k).suc)))
    is_pell_solution(Int.2,
        Int.from_nat(continued_fraction_convergent_numerator(
            sqrt_two_continued_fraction_coefficients, (Nat.2 * k).suc)),
        Int.from_nat(continued_fraction_convergent_denominator(
            sqrt_two_continued_fraction_coefficients, (Nat.2 * k).suc)))
}

/// Every convergent of √2 yields a solution of Pell's equation: either the
/// convergent itself, or its successor obtained by composing with the
/// fundamental unit 1 + √2, solves x² - 2·y² = 1.
theorem cf_pell_each_convergent_yields_pell_solution(n: Nat) {
    is_pell_solution(Int.2,
        Int.from_nat(continued_fraction_convergent_numerator(
            sqrt_two_continued_fraction_coefficients, n)),
        Int.from_nat(continued_fraction_convergent_denominator(
            sqrt_two_continued_fraction_coefficients, n))) or
    is_pell_solution(Int.2,
        Int.from_nat(continued_fraction_convergent_numerator(
            sqrt_two_continued_fraction_coefficients, n.suc)),
        Int.from_nat(continued_fraction_convergent_denominator(
            sqrt_two_continued_fraction_coefficients, n.suc)))
} by {
    cf_pell_convergent_norm_one_or_neg_one(n)
    pell_norm(Int.2,
        Int.from_nat(continued_fraction_convergent_numerator(
            sqrt_two_continued_fraction_coefficients, n)),
        Int.from_nat(continued_fraction_convergent_denominator(
            sqrt_two_continued_fraction_coefficients, n))) = Int.1 or
        pell_norm(Int.2,
            Int.from_nat(continued_fraction_convergent_numerator(
                sqrt_two_continued_fraction_coefficients, n)),
            Int.from_nat(continued_fraction_convergent_denominator(
                sqrt_two_continued_fraction_coefficients, n))) = -Int.1
    if pell_norm(Int.2,
            Int.from_nat(continued_fraction_convergent_numerator(
                sqrt_two_continued_fraction_coefficients, n)),
            Int.from_nat(continued_fraction_convergent_denominator(
                sqrt_two_continued_fraction_coefficients, n))) = Int.1 {
        pell_norm(Int.2,
            Int.from_nat(continued_fraction_convergent_numerator(
                sqrt_two_continued_fraction_coefficients, n)),
            Int.from_nat(continued_fraction_convergent_denominator(
                sqrt_two_continued_fraction_coefficients, n))) = Int.1
        is_pell_solution(Int.2,
            Int.from_nat(continued_fraction_convergent_numerator(
                sqrt_two_continued_fraction_coefficients, n)),
            Int.from_nat(continued_fraction_convergent_denominator(
                sqrt_two_continued_fraction_coefficients, n)))
        is_pell_solution(Int.2,
            Int.from_nat(continued_fraction_convergent_numerator(
                sqrt_two_continued_fraction_coefficients, n)),
            Int.from_nat(continued_fraction_convergent_denominator(
                sqrt_two_continued_fraction_coefficients, n))) or
            is_pell_solution(Int.2,
                Int.from_nat(continued_fraction_convergent_numerator(
                    sqrt_two_continued_fraction_coefficients, n.suc)),
                Int.from_nat(continued_fraction_convergent_denominator(
                    sqrt_two_continued_fraction_coefficients, n.suc)))
    }
    if pell_norm(Int.2,
            Int.from_nat(continued_fraction_convergent_numerator(
                sqrt_two_continued_fraction_coefficients, n)),
            Int.from_nat(continued_fraction_convergent_denominator(
                sqrt_two_continued_fraction_coefficients, n))) = -Int.1 {
        cf_pell_convergent_norm_alternating(n.suc)
        cf_pell_convergent_norm_alternating(n)
        pell_norm(Int.2,
            Int.from_nat(continued_fraction_convergent_numerator(
                sqrt_two_continued_fraction_coefficients, n.suc)),
            Int.from_nat(continued_fraction_convergent_denominator(
                sqrt_two_continued_fraction_coefficients, n.suc))) =
            alternating_sign[Int](n.suc.suc)
        pell_norm(Int.2,
            Int.from_nat(continued_fraction_convergent_numerator(
                sqrt_two_continued_fraction_coefficients, n)),
            Int.from_nat(continued_fraction_convergent_denominator(
                sqrt_two_continued_fraction_coefficients, n))) =
            alternating_sign[Int](n.suc)
        alternating_sign_suc[Int](n.suc)
        alternating_sign[Int](n.suc.suc) = -alternating_sign[Int](n.suc)
        pell_norm(Int.2,
            Int.from_nat(continued_fraction_convergent_numerator(
                sqrt_two_continued_fraction_coefficients, n.suc)),
            Int.from_nat(continued_fraction_convergent_denominator(
                sqrt_two_continued_fraction_coefficients, n.suc))) =
            -pell_norm(Int.2,
                Int.from_nat(continued_fraction_convergent_numerator(
                    sqrt_two_continued_fraction_coefficients, n)),
                Int.from_nat(continued_fraction_convergent_denominator(
                    sqrt_two_continued_fraction_coefficients, n)))
        pell_norm(Int.2,
            Int.from_nat(continued_fraction_convergent_numerator(
                sqrt_two_continued_fraction_coefficients, n.suc)),
            Int.from_nat(continued_fraction_convergent_denominator(
                sqrt_two_continued_fraction_coefficients, n.suc))) = -(-Int.1)
        --Int.1 = Int.1
        pell_norm(Int.2,
            Int.from_nat(continued_fraction_convergent_numerator(
                sqrt_two_continued_fraction_coefficients, n.suc)),
            Int.from_nat(continued_fraction_convergent_denominator(
                sqrt_two_continued_fraction_coefficients, n.suc))) = Int.1
        is_pell_solution(Int.2,
            Int.from_nat(continued_fraction_convergent_numerator(
                sqrt_two_continued_fraction_coefficients, n.suc)),
            Int.from_nat(continued_fraction_convergent_denominator(
                sqrt_two_continued_fraction_coefficients, n.suc)))
        is_pell_solution(Int.2,
            Int.from_nat(continued_fraction_convergent_numerator(
                sqrt_two_continued_fraction_coefficients, n)),
            Int.from_nat(continued_fraction_convergent_denominator(
                sqrt_two_continued_fraction_coefficients, n))) or
            is_pell_solution(Int.2,
                Int.from_nat(continued_fraction_convergent_numerator(
                    sqrt_two_continued_fraction_coefficients, n.suc)),
                Int.from_nat(continued_fraction_convergent_denominator(
                    sqrt_two_continued_fraction_coefficients, n.suc)))
    }
    is_pell_solution(Int.2,
        Int.from_nat(continued_fraction_convergent_numerator(
            sqrt_two_continued_fraction_coefficients, n)),
        Int.from_nat(continued_fraction_convergent_denominator(
            sqrt_two_continued_fraction_coefficients, n))) or
        is_pell_solution(Int.2,
            Int.from_nat(continued_fraction_convergent_numerator(
                sqrt_two_continued_fraction_coefficients, n.suc)),
            Int.from_nat(continued_fraction_convergent_denominator(
                sqrt_two_continued_fraction_coefficients, n.suc)))
}

// ============================================================================
// Section 5: the fundamental unit
// ============================================================================

/// Composing a pair of norm minus one with the fundamental unit 1 + √2 gives
/// a solution of Pell's equation: from x² - 2·y² = -1 follows
/// (x + 2·y)² - 2·(x + y)² = 1.
theorem cf_pell_norm_neg_one_compose_fundamental(x: Int, y: Int) {
    pell_norm(Int.2, x, y) = -Int.1 implies
        is_pell_solution(Int.2, x + Int.2 * y, x + y)
} by {
    if pell_norm(Int.2, x, y) = -Int.1 {
        cf_pell_norm_flip_compose_one(x, y)
        pell_norm(Int.2, x + Int.2 * y, x + y) = -pell_norm(Int.2, x, y)
        pell_norm(Int.2, x + Int.2 * y, x + y) = -(-Int.1)
        --Int.1 = Int.1
        pell_norm(Int.2, x + Int.2 * y, x + y) = Int.1
        pell_norm(Int.2, x + Int.2 * y, x + y) =
            (x + Int.2 * y) * (x + Int.2 * y) - Int.2 * ((x + y) * (x + y))
        is_pell_solution(Int.2, x + Int.2 * y, x + y)
    }
}

/// The composition of the fundamental unit with a pair, expressed as two
/// compositions with 1 + √2: 3·x + 4·y = (x + 2·y) + 2·(x + y).
theorem cf_pell_fundamental_unit_compose_first(x: Int, y: Int) {
    Int.3 * x + Int.4 * y = (x + Int.2 * y) + Int.2 * (x + y)
} by {
    Int.2 * (x + y) = Int.2 * x + Int.2 * y
    x + Int.2 * x = Int.1 * x + Int.2 * x
    Int.1 * x + Int.2 * x = (Int.1 + Int.2) * x
    add_from_nat(Nat.1, Nat.2)
    Int.from_nat(Nat.1) + Int.from_nat(Nat.2) = Int.from_nat(Nat.3)
    Int.from_nat(Nat.1) = Int.1
    Int.from_nat(Nat.2) = Int.2
    Int.from_nat(Nat.3) = Int.3
    Int.1 + Int.2 = Int.3
    (Int.1 + Int.2) * x = Int.3 * x
    x + Int.2 * x = Int.3 * x
    Int.2 * y + Int.2 * y = Int.2 * (y + y)
    y + y = Int.2 * y
    Int.2 * (y + y) = Int.2 * (Int.2 * y)
    Int.2 * (Int.2 * y) = (Int.2 * Int.2) * y
    mul_from_nat(Nat.2, Nat.2)
    Int.from_nat(Nat.2) * Int.from_nat(Nat.2) = Int.from_nat(Nat.4)
    Int.from_nat(Nat.2) = Int.2
    Int.from_nat(Nat.4) = Int.4
    Int.2 * Int.2 = Int.4
    (Int.2 * Int.2) * y = Int.4 * y
    Int.2 * (Int.2 * y) = Int.4 * y
    Int.2 * y + Int.2 * y = Int.4 * y
    (x + Int.2 * y) + (Int.2 * x + Int.2 * y) =
        (x + Int.2 * x) + (Int.2 * y + Int.2 * y)
    (x + Int.2 * x) + (Int.2 * y + Int.2 * y) = Int.3 * x + Int.4 * y
    (x + Int.2 * y) + Int.2 * (x + y) =
        (x + Int.2 * y) + (Int.2 * x + Int.2 * y)
    (x + Int.2 * y) + Int.2 * (x + y) = Int.3 * x + Int.4 * y
    Int.3 * x + Int.4 * y = (x + Int.2 * y) + Int.2 * (x + y)
}

/// The composition of the fundamental unit with a pair, expressed as two
/// compositions with 1 + √2: 2·x + 3·y = (x + 2·y) + (x + y).
theorem cf_pell_fundamental_unit_compose_second(x: Int, y: Int) {
    Int.2 * x + Int.3 * y = (x + Int.2 * y) + (x + y)
} by {
    (x + Int.2 * y) + (x + y) = (x + x) + (Int.2 * y + y)
    x + x = Int.2 * x
    Int.2 * y + y = Int.2 * y + Int.1 * y
    Int.2 * y + Int.1 * y = (Int.2 + Int.1) * y
    add_from_nat(Nat.2, Nat.1)
    Int.from_nat(Nat.2) + Int.from_nat(Nat.1) = Int.from_nat(Nat.3)
    Int.from_nat(Nat.2) = Int.2
    Int.from_nat(Nat.1) = Int.1
    Int.from_nat(Nat.3) = Int.3
    Int.2 + Int.1 = Int.3
    (Int.2 + Int.1) * y = Int.3 * y
    Int.2 * y + y = Int.3 * y
    (x + x) + (Int.2 * y + y) = Int.2 * x + Int.3 * y
    (x + Int.2 * y) + (x + y) = Int.2 * x + Int.3 * y
    Int.2 * x + Int.3 * y = (x + Int.2 * y) + (x + y)
}

/// Composing with the fundamental unit (3, 2) = (1 + √2)² preserves the Pell
/// norm: (3·x + 4·y)² - 2·(2·x + 3·y)² = x² - 2·y². This is two applications
/// of the norm flip: (3, 2) = (1, 1) ⊙ (1, 1) with the composition step.
theorem cf_pell_fundamental_unit_preserves_norm(x: Int, y: Int) {
    pell_norm(Int.2, Int.3 * x + Int.4 * y, Int.2 * x + Int.3 * y) =
        pell_norm(Int.2, x, y)
} by {
    cf_pell_fundamental_unit_compose_first(x, y)
    cf_pell_fundamental_unit_compose_second(x, y)
    Int.3 * x + Int.4 * y = (x + Int.2 * y) + Int.2 * (x + y)
    Int.2 * x + Int.3 * y = (x + Int.2 * y) + (x + y)
    pell_norm(Int.2, Int.3 * x + Int.4 * y, Int.2 * x + Int.3 * y) =
        pell_norm(Int.2, (x + Int.2 * y) + Int.2 * (x + y), (x + Int.2 * y) + (x + y))
    cf_pell_norm_flip_compose_one(x + Int.2 * y, x + y)
    pell_norm(Int.2, (x + Int.2 * y) + Int.2 * (x + y), (x + Int.2 * y) + (x + y)) =
        -pell_norm(Int.2, x + Int.2 * y, x + y)
    pell_norm(Int.2, Int.3 * x + Int.4 * y, Int.2 * x + Int.3 * y) =
        -pell_norm(Int.2, x + Int.2 * y, x + y)
    cf_pell_norm_flip_compose_one(x, y)
    pell_norm(Int.2, x + Int.2 * y, x + y) = -pell_norm(Int.2, x, y)
    pell_norm(Int.2, Int.3 * x + Int.4 * y, Int.2 * x + Int.3 * y) =
        -(-pell_norm(Int.2, x, y))
    neg_neg(pell_norm(Int.2, x, y))
    -(-pell_norm(Int.2, x, y)) = pell_norm(Int.2, x, y)
    pell_norm(Int.2, Int.3 * x + Int.4 * y, Int.2 * x + Int.3 * y) = pell_norm(Int.2, x, y)
}

/// Composing any Pell solution with the fundamental unit (3, 2) = (1 + √2)²
/// gives another Pell solution: (3·x + 4·y, 2·x + 3·y) solves x² - 2·y² = 1.
theorem cf_pell_fundamental_unit_preserves_pell_solution(x: Int, y: Int) {
    is_pell_solution(Int.2, x, y) implies
        is_pell_solution(Int.2, Int.3 * x + Int.4 * y, Int.2 * x + Int.3 * y)
} by {
    if is_pell_solution(Int.2, x, y) {
        pell_norm(Int.2, x, y) = Int.1
        cf_pell_fundamental_unit_preserves_norm(x, y)
        pell_norm(Int.2, Int.3 * x + Int.4 * y, Int.2 * x + Int.3 * y) =
            pell_norm(Int.2, x, y)
        pell_norm(Int.2, Int.3 * x + Int.4 * y, Int.2 * x + Int.3 * y) = Int.1
        is_pell_solution(Int.2, Int.3 * x + Int.4 * y, Int.2 * x + Int.3 * y)
    }
}

/// The fundamental solution (3, 2) of x² - 2·y² = 1 is the first convergent
/// of the continued fraction of √2 (pell.ac), and every odd convergent
/// continues the sequence of solutions generated from it.
theorem cf_pell_fundamental_solution_is_first_convergent {
    is_pell_solution(Int.2,
        Int.from_nat(continued_fraction_convergent_numerator(
            sqrt_two_continued_fraction_coefficients, Nat.1)),
        Int.from_nat(continued_fraction_convergent_denominator(
            sqrt_two_continued_fraction_coefficients, Nat.1)))
} by {
    pell_two_fundamental_solution_from_convergents
}

// ============================================================================
// Section 6: the general bound (statement)
// ============================================================================

// The classical estimate: if p/q is a convergent of the continued fraction of
// √d (d a nonsquare positive integer), then |p² - d·q²| < 2·√d + 1.
//
// The algebraic core is available: the adjacent-determinant identity
// p_{n+1}·q_n - p_n·q_{n+1} = ±1 and the denominator growth bounds live in
// continued_fraction_convergents.ac and continued_fraction_approx.ac. What is
// missing is the real-analysis statement that the continued fraction of √d
// converges to √d as a real number (the library formalizes convergent gaps
// but not the specific limit √d), so the general estimate is stated here but
// not proved, as in pell.ac.
//
// For d = 2 the exact instance is proved in Section 3:
// |p_n² - 2·q_n²| = 1 for every convergent of √2, which is the sharpest
// possible value of the bound.

// theorem cf_pell_convergent_error_bound(d: Nat, p: Nat, q: Nat) {
//     ... |p² - d·q²| < 2·√d + 1 for convergents of the continued fraction of √d ...
// }
