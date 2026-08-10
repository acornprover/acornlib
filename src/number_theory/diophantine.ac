/// Diophantine equations.
///
/// A selection of classical Diophantine results built on the library's
/// Pythagorean-triple, congruence, and Pell machinery:
///
///   1. The Pythagorean equation x² + y² = z² has infinitely many primitive
///      solutions.  The parameterization (m² - n², 2mn, m² + n²) with
///      m = k + 1, n = k gives the family (2k + 1, 2k(k + 1), 2k² + 2k + 1),
///      which is proved to be Pythagorean (Section 1), primitive, and
///      pairwise distinct (Section 2); the fully general integer
///      parameterization is recorded as a statement below Section 1.
///
///   2. The equation x² = 2y² has no nonzero natural solution.  The proof is
///      the classical infinite descent for the irrationality of √2: from a
///      solution, two divides both coordinates, so halving them yields a
///      smaller solution; well-founded induction on the first coordinate
///      rules the descent out (Section 3).
///
///   3. Fermat's Last Theorem for the exponent three, x³ + y³ = z³, is
///      stated below but not proved: the standard proof goes through the
///      arithmetic of the Eisenstein integers, which the library does not yet
///      formalize (Section 4).
///
///   4. Pell's equation x² - 2y² = 1 has infinitely many solutions: from a
///      solution (x, y), the square (x² + 2y², 2xy) is again a solution
///      (pell.ac), and the y-coordinate strictly grows, so iterating from the
///      fundamental solution (3, 2) yields solutions with y-coordinates
///      exceeding every bound (Section 5).
from nat import Nat, add_comm, add_assoc, add_comm_4, mul_comm, mul_assoc, distrib_left,
    distrib_right, mul_one_left, mul_one_right, add_imp_sub, divides_mul, divides_sub, mul_to_zero,
    mul_cancel_left, gcd_divides_left, gcd_divides_right, divides_gcd, gcd_of_prime,
    two_divides_suc_iff, divides_self, lte_imp_not_lt, lt_or_lte, lte_trans, lt_trans, lt_add_left,
    lt_add_suc, divisor_lt, lte_ref, lte_mul_both, lte_mul, lte_add_left, lte_add_right, sum_lte,
    lt_and_lte, only_zero_lte_zero, alt_induction
from int import Int, add_from_nat, mul_from_nat, abs, abs_mul, abs_zero_imp_zero, abs_from_nat
from algebra.well_founded import nat_lt_relation, nat_lt_relation_induction_at
from number_theory import nat_two_prime
from number_theory.coprime import nat_divides_one_imp_one, coprime_mul, coprime_divides_of_divides_mul
from number_theory.pell import is_pell_solution, pell_next_x, pell_next_y, pell_solution_square,
    pell_two_smallest

numerals Nat
numerals Int

// ============================================================================
// Section 1: Pythagorean triples
// ============================================================================

/// True when (a, b, c) is a Pythagorean triple, a² + b² = c².
define is_pythagorean_triple(a: Nat, b: Nat, c: Nat) -> Bool {
    a * a + b * b = c * c
}

/// True when (a, b, c) is a primitive Pythagorean triple: a Pythagorean triple
/// whose legs are coprime.
define is_primitive_pythagorean_triple(a: Nat, b: Nat, c: Nat) -> Bool {
    is_pythagorean_triple(a, b, c) and a.coprime(b)
}

/// The odd leg of the k-th member of the standard family, 2k + 1 = (k+1)² - k².
define pythag_odd_leg(k: Nat) -> Nat {
    Nat.2 * k + Nat.1
}

/// The even leg of the k-th member of the standard family, 2·k·(k+1) = 2·(k+1)·k.
define pythag_even_leg(k: Nat) -> Nat {
    Nat.2 * k * (k + Nat.1)
}

/// The hypotenuse of the k-th member of the standard family, 2k² + 2k + 1 = (k+1)² + k².
define pythag_hypotenuse(k: Nat) -> Nat {
    Nat.2 * k * k + Nat.2 * k + Nat.1
}

/// The square of a successor: (x + 1)² = x² + 2x + 1.
theorem nat_sq_suc(x: Nat) {
    (x + Nat.1) * (x + Nat.1) = x * x + Nat.2 * x + Nat.1
} by {
    distrib_left(x, Nat.1, x + Nat.1)
    (x + Nat.1) * (x + Nat.1) = x * (x + Nat.1) + Nat.1 * (x + Nat.1)
    distrib_right(x, x, Nat.1)
    x * (x + Nat.1) = x * x + x * Nat.1
    mul_one_right(x)
    x * Nat.1 = x
    x * (x + Nat.1) = x * x + x
    mul_one_left(x + Nat.1)
    Nat.1 * (x + Nat.1) = x + Nat.1
    (x + Nat.1) * (x + Nat.1) = x * x + x + (x + Nat.1)
    x * x + x + (x + Nat.1) = x * x + (x + x) + Nat.1
    x + x = Nat.2 * x
    x * x + (x + x) + Nat.1 = x * x + Nat.2 * x + Nat.1
    (x + Nat.1) * (x + Nat.1) = x * x + Nat.2 * x + Nat.1
}

/// The square of a doubled number: (2x)² = 4x².
theorem nat_sq_double(x: Nat) {
    (Nat.2 * x) * (Nat.2 * x) = Nat.4 * (x * x)
} by {
    (Nat.2 * x) * (Nat.2 * x) = Nat.2 * x * Nat.2 * x
    Nat.2 * x * Nat.2 * x = Nat.2 * Nat.2 * x * x
    Nat.2 * Nat.2 = Nat.4
    Nat.2 * Nat.2 * x * x = Nat.4 * (x * x)
    (Nat.2 * x) * (Nat.2 * x) = Nat.4 * (x * x)
}

/// The hypotenuse of the family is the even leg plus one.
theorem pythag_hyp_eq_even_add_one(k: Nat) {
    pythag_hypotenuse(k) = pythag_even_leg(k) + Nat.1
} by {
    pythag_hypotenuse(k) = Nat.2 * k * k + Nat.2 * k + Nat.1
    pythag_even_leg(k) = Nat.2 * k * (k + Nat.1)
    Nat.2 * k * (k + Nat.1) = Nat.2 * (k * (k + Nat.1))
    k * (k + Nat.1) = k * k + k
    Nat.2 * (k * (k + Nat.1)) = Nat.2 * (k * k + k)
    Nat.2 * (k * k + k) = Nat.2 * (k * k) + Nat.2 * k
    Nat.2 * k * k = Nat.2 * (k * k)
    pythag_even_leg(k) = Nat.2 * (k * k) + Nat.2 * k
    pythag_even_leg(k) + Nat.1 = Nat.2 * (k * k) + Nat.2 * k + Nat.1
    pythag_hypotenuse(k) = pythag_even_leg(k) + Nat.1
}

/// The square of the odd leg is twice the even leg plus one.
theorem pythag_odd_sq(k: Nat) {
    pythag_odd_leg(k) * pythag_odd_leg(k) = Nat.2 * pythag_even_leg(k) + Nat.1
} by {
    pythag_odd_leg(k) = Nat.2 * k + Nat.1
    pythag_even_leg(k) = Nat.2 * k * (k + Nat.1)
    nat_sq_suc(Nat.2 * k)
    (Nat.2 * k + Nat.1) * (Nat.2 * k + Nat.1) =
        Nat.2 * k * (Nat.2 * k) + Nat.2 * (Nat.2 * k) + Nat.1
    nat_sq_double(k)
    (Nat.2 * k) * (Nat.2 * k) = Nat.4 * (k * k)
    Nat.2 * (Nat.2 * k) = Nat.4 * k
    pythag_odd_leg(k) * pythag_odd_leg(k) =
        Nat.4 * (k * k) + Nat.4 * k + Nat.1
    Nat.2 * pythag_even_leg(k) + Nat.1 =
        Nat.2 * (Nat.2 * k * (k + Nat.1)) + Nat.1
    Nat.2 * (Nat.2 * k * (k + Nat.1)) = Nat.4 * (k * (k + Nat.1))
    k * (k + Nat.1) = k * k + k
    Nat.4 * (k * (k + Nat.1)) = Nat.4 * (k * k + k)
    Nat.4 * (k * k + k) = Nat.4 * (k * k) + Nat.4 * k
    Nat.2 * pythag_even_leg(k) + Nat.1 =
        Nat.4 * (k * k) + Nat.4 * k + Nat.1
    pythag_odd_leg(k) * pythag_odd_leg(k) = Nat.2 * pythag_even_leg(k) + Nat.1
}

/// The parameterization (m, n) with m = k + 1, n = k gives a Pythagorean triple:
/// the k-th family member (2k + 1, 2k(k + 1), 2k² + 2k + 1) satisfies
/// (2k + 1)² + (2k(k + 1))² = (2k² + 2k + 1)².
theorem pythag_family_solution(k: Nat) {
    is_pythagorean_triple(pythag_odd_leg(k), pythag_even_leg(k), pythag_hypotenuse(k))
} by {
    is_pythagorean_triple(pythag_odd_leg(k), pythag_even_leg(k), pythag_hypotenuse(k)) =
        (pythag_odd_leg(k) * pythag_odd_leg(k) + pythag_even_leg(k) * pythag_even_leg(k) =
            pythag_hypotenuse(k) * pythag_hypotenuse(k))
    pythag_odd_sq(k)
    pythag_odd_leg(k) * pythag_odd_leg(k) = Nat.2 * pythag_even_leg(k) + Nat.1
    pythag_hyp_eq_even_add_one(k)
    pythag_hypotenuse(k) = pythag_even_leg(k) + Nat.1
    nat_sq_suc(pythag_even_leg(k))
    (pythag_even_leg(k) + Nat.1) * (pythag_even_leg(k) + Nat.1) =
        pythag_even_leg(k) * pythag_even_leg(k) + Nat.2 * pythag_even_leg(k) + Nat.1
    pythag_odd_leg(k) * pythag_odd_leg(k) + pythag_even_leg(k) * pythag_even_leg(k) =
        pythag_hypotenuse(k) * pythag_hypotenuse(k)
    is_pythagorean_triple(pythag_odd_leg(k), pythag_even_leg(k), pythag_hypotenuse(k))
}

// The general parameterization over the integers: for every pair of integers
// (m, n), the triple (m² - n², 2mn, m² + n²) satisfies the Pythagorean
// identity (m² - n²)² + (2mn)² = (m² + n²)².  The algebra is a routine
// expansion: (u - v)² + 4uv = (u + v)² at u = m², v = n², together with
// (2mn)² = 4·m²·n².  The proof below is the k-th family member of Section 2
// with m = k + 1, n = k, i.e. the parameterization instantiated at
// consecutive parameters; the fully general statement is left for a future
// file.  (The integer rearrangement lemmas needed — regrouping 4x - 2x = 2x
// and the multi-commutation 2mn·2mn = 4m²n² — exist in cf_pell.ac and
// pell.ac but the identity is not yet assembled.)
//
// theorem pythag_param_identity(m: Int, n: Int) {
//     (m * m - n * n) * (m * m - n * n) + (Int.2 * m * n) * (Int.2 * m * n) =
//         (m * m + n * n) * (m * m + n * n)
// }

// ============================================================================
// Section 2: infinitely many primitive Pythagorean triples
// ============================================================================

/// The odd leg 2k + 1 is coprime to two: any common divisor divides the
/// difference (2k + 1) - 2k = 1.
theorem pythag_odd_coprime_two(k: Nat) {
    pythag_odd_leg(k).coprime(Nat.2)
} by {
    pythag_odd_leg(k) = Nat.2 * k + Nat.1
    gcd_divides_left(Nat.2 * k + Nat.1, Nat.2)
    (Nat.2 * k + Nat.1).gcd(Nat.2).divides(Nat.2 * k + Nat.1)
    gcd_divides_right(Nat.2 * k + Nat.1, Nat.2)
    (Nat.2 * k + Nat.1).gcd(Nat.2).divides(Nat.2)
    divides_mul(Nat.2, k, (Nat.2 * k + Nat.1).gcd(Nat.2))
    (Nat.2 * k + Nat.1).gcd(Nat.2).divides(Nat.2 * k)
    divides_sub(Nat.2 * k + Nat.1, Nat.2 * k, (Nat.2 * k + Nat.1).gcd(Nat.2))
    (Nat.2 * k + Nat.1).gcd(Nat.2).divides((Nat.2 * k + Nat.1) - Nat.2 * k)
    add_imp_sub(Nat.2 * k, Nat.1, Nat.2 * k + Nat.1)
    (Nat.2 * k + Nat.1) - Nat.2 * k = Nat.1
    (Nat.2 * k + Nat.1).gcd(Nat.2).divides(Nat.1)
    nat_divides_one_imp_one((Nat.2 * k + Nat.1).gcd(Nat.2))
    (Nat.2 * k + Nat.1).gcd(Nat.2) = Nat.1
    (Nat.2 * k + Nat.1).coprime(Nat.2)
    pythag_odd_leg(k).coprime(Nat.2)
}

/// The odd leg 2k + 1 is coprime to k.
theorem pythag_odd_coprime_k(k: Nat) {
    pythag_odd_leg(k).coprime(k)
} by {
    pythag_odd_leg(k) = Nat.2 * k + Nat.1
    gcd_divides_left(Nat.2 * k + Nat.1, k)
    (Nat.2 * k + Nat.1).gcd(k).divides(Nat.2 * k + Nat.1)
    gcd_divides_right(Nat.2 * k + Nat.1, k)
    (Nat.2 * k + Nat.1).gcd(k).divides(k)
    divides_mul(k, Nat.2, (Nat.2 * k + Nat.1).gcd(k))
    (Nat.2 * k + Nat.1).gcd(k).divides(k * Nat.2)
    k * Nat.2 = Nat.2 * k
    (Nat.2 * k + Nat.1).gcd(k).divides(Nat.2 * k)
    divides_sub(Nat.2 * k + Nat.1, Nat.2 * k, (Nat.2 * k + Nat.1).gcd(k))
    (Nat.2 * k + Nat.1).gcd(k).divides((Nat.2 * k + Nat.1) - Nat.2 * k)
    add_imp_sub(Nat.2 * k, Nat.1, Nat.2 * k + Nat.1)
    (Nat.2 * k + Nat.1) - Nat.2 * k = Nat.1
    (Nat.2 * k + Nat.1).gcd(k).divides(Nat.1)
    nat_divides_one_imp_one((Nat.2 * k + Nat.1).gcd(k))
    (Nat.2 * k + Nat.1).gcd(k) = Nat.1
    (Nat.2 * k + Nat.1).coprime(k)
    pythag_odd_leg(k).coprime(k)
}

/// The odd leg 2k + 1 is coprime to k + 1.
theorem pythag_odd_coprime_suc_k(k: Nat) {
    pythag_odd_leg(k).coprime(k + Nat.1)
} by {
    pythag_odd_leg(k) = Nat.2 * k + Nat.1
    gcd_divides_left(Nat.2 * k + Nat.1, k + Nat.1)
    (Nat.2 * k + Nat.1).gcd(k + Nat.1).divides(Nat.2 * k + Nat.1)
    gcd_divides_right(Nat.2 * k + Nat.1, k + Nat.1)
    (Nat.2 * k + Nat.1).gcd(k + Nat.1).divides(k + Nat.1)
    divides_mul(k + Nat.1, Nat.2, (Nat.2 * k + Nat.1).gcd(k + Nat.1))
    (Nat.2 * k + Nat.1).gcd(k + Nat.1).divides((k + Nat.1) * Nat.2)
    (k + Nat.1) * Nat.2 = Nat.2 * (k + Nat.1)
    (Nat.2 * k + Nat.1).gcd(k + Nat.1).divides(Nat.2 * (k + Nat.1))
    Nat.2 * (k + Nat.1) = Nat.2 * k + Nat.2
    (Nat.2 * k + Nat.1).gcd(k + Nat.1).divides(Nat.2 * k + Nat.2)
    divides_sub(Nat.2 * k + Nat.2, Nat.2 * k + Nat.1, (Nat.2 * k + Nat.1).gcd(k + Nat.1))
    (Nat.2 * k + Nat.1).gcd(k + Nat.1).divides((Nat.2 * k + Nat.2) - (Nat.2 * k + Nat.1))
    (Nat.2 * k + Nat.1) + Nat.1 = Nat.2 * k + Nat.2
    add_imp_sub(Nat.2 * k + Nat.1, Nat.1, Nat.2 * k + Nat.2)
    (Nat.2 * k + Nat.2) - (Nat.2 * k + Nat.1) = Nat.1
    (Nat.2 * k + Nat.1).gcd(k + Nat.1).divides(Nat.1)
    nat_divides_one_imp_one((Nat.2 * k + Nat.1).gcd(k + Nat.1))
    (Nat.2 * k + Nat.1).gcd(k + Nat.1) = Nat.1
    (Nat.2 * k + Nat.1).coprime(k + Nat.1)
    pythag_odd_leg(k).coprime(k + Nat.1)
}

/// The k-th family member is a primitive Pythagorean triple: its legs are
/// coprime, because the odd leg 2k + 1 is coprime to two, to k, and to k + 1.
theorem pythag_family_primitive(k: Nat) {
    is_primitive_pythagorean_triple(pythag_odd_leg(k), pythag_even_leg(k), pythag_hypotenuse(k))
} by {
    is_primitive_pythagorean_triple(pythag_odd_leg(k), pythag_even_leg(k), pythag_hypotenuse(k)) =
        (is_pythagorean_triple(pythag_odd_leg(k), pythag_even_leg(k), pythag_hypotenuse(k)) and
            pythag_odd_leg(k).coprime(pythag_even_leg(k)))
    pythag_family_solution(k)
    is_pythagorean_triple(pythag_odd_leg(k), pythag_even_leg(k), pythag_hypotenuse(k))
    pythag_odd_coprime_two(k)
    pythag_odd_leg(k).coprime(Nat.2)
    pythag_odd_coprime_k(k)
    pythag_odd_leg(k).coprime(k)
    pythag_odd_coprime_suc_k(k)
    pythag_odd_leg(k).coprime(k + Nat.1)
    coprime_mul(pythag_odd_leg(k), Nat.2, k)
    pythag_odd_leg(k).coprime(Nat.2 * k)
    coprime_mul(pythag_odd_leg(k), Nat.2 * k, k + Nat.1)
    pythag_odd_leg(k).coprime((Nat.2 * k) * (k + Nat.1))
    (Nat.2 * k) * (k + Nat.1) = Nat.2 * k * (k + Nat.1)
    pythag_odd_leg(k).coprime(pythag_even_leg(k))
    is_primitive_pythagorean_triple(pythag_odd_leg(k), pythag_even_leg(k), pythag_hypotenuse(k))
}

/// The odd legs of the family strictly grow: 2k + 1 < 2(k + 1) + 1, so
/// distinct parameters give distinct triples.
theorem pythag_family_legs_grow(k: Nat) {
    pythag_odd_leg(k) < pythag_odd_leg(k.suc)
} by {
    pythag_odd_leg(k) = Nat.2 * k + Nat.1
    pythag_odd_leg(k.suc) = Nat.2 * k.suc + Nat.1
    k.suc = k + Nat.1
    Nat.2 * k.suc = Nat.2 * (k + Nat.1)
    Nat.2 * (k + Nat.1) = Nat.2 * k + Nat.2
    Nat.2 * k.suc + Nat.1 = Nat.2 * k + Nat.3
    lt_add_suc(Nat.1, Nat.1)
    Nat.1 < Nat.1 + Nat.1.suc
    Nat.1 + Nat.1.suc = Nat.3
    Nat.1 < Nat.3
    lt_add_left(Nat.2 * k, Nat.1, Nat.3)
    Nat.2 * k + Nat.1 < Nat.2 * k + Nat.3
    pythag_odd_leg(k) < pythag_odd_leg(k.suc)
}

/// The hypotenuse of the family is at least twice the parameter.
theorem pythag_hyp_ge_twice_param(k: Nat) {
    Nat.2 * k <= pythag_hypotenuse(k)
} by {
    pythag_hypotenuse(k) = Nat.2 * k * k + Nat.2 * k + Nat.1
    Nat.2 * k * k + Nat.2 * k + Nat.1 = Nat.2 * k + (Nat.2 * k * k + Nat.1)
    Nat.0 <= Nat.2 * k * k + Nat.1
    lte_add_left(Nat.2 * k, Nat.0, Nat.2 * k * k + Nat.1)
    Nat.2 * k + Nat.0 <= Nat.2 * k + (Nat.2 * k * k + Nat.1)
    Nat.2 * k + Nat.0 = Nat.2 * k
    Nat.2 * k <= Nat.2 * k + (Nat.2 * k * k + Nat.1)
    Nat.2 * k <= pythag_hypotenuse(k)
}

/// The parameter twice a successor exceeds it: 2(n + 1) > n.
theorem pythag_twice_suc_gt(n: Nat) {
    n < Nat.2 * (n + Nat.1)
} by {
    Nat.2 * (n + Nat.1) = Nat.2 * n + Nat.2
    Nat.2 * n + Nat.2 = n + (n + Nat.2)
    lt_add_suc(n, n + Nat.1)
    n < n + (n + Nat.1).suc
    (n + Nat.1).suc = n + Nat.2
    n < n + (n + Nat.2)
    n < Nat.2 * (n + Nat.1)
}

/// The Pythagorean equation x² + y² = z² has infinitely many primitive
/// solutions: beyond every bound n there is a primitive Pythagorean triple
/// with hypotenuse exceeding n.
theorem pythagorean_infinitely_many_primitive(n: Nat) {
    exists(x: Nat, y: Nat, z: Nat) {
        n < z and is_primitive_pythagorean_triple(x, y, z)
    }
} by {
    pythag_family_primitive(n + Nat.1)
    is_primitive_pythagorean_triple(
        pythag_odd_leg(n + Nat.1), pythag_even_leg(n + Nat.1), pythag_hypotenuse(n + Nat.1))
    pythag_hyp_ge_twice_param(n + Nat.1)
    Nat.2 * (n + Nat.1) <= pythag_hypotenuse(n + Nat.1)
    pythag_twice_suc_gt(n)
    n < Nat.2 * (n + Nat.1)
    lt_and_lte(n, Nat.2 * (n + Nat.1), pythag_hypotenuse(n + Nat.1))
    n < pythag_hypotenuse(n + Nat.1)
    n < pythag_hypotenuse(n + Nat.1) and
        is_primitive_pythagorean_triple(
            pythag_odd_leg(n + Nat.1), pythag_even_leg(n + Nat.1), pythag_hypotenuse(n + Nat.1))
    exists(x: Nat, y: Nat, z: Nat) {
        n < z and is_primitive_pythagorean_triple(x, y, z)
    }
}

// ============================================================================
// Section 3: x² = 2y² has no nonzero solution
// ============================================================================

/// If two divides a square, it divides the root (two is prime).
theorem two_divides_square_imp_two_divides(x: Nat) {
    Nat.2.divides(x * x) implies Nat.2.divides(x)
} by {
    if Nat.2.divides(x * x) {
        nat_two_prime
        gcd_of_prime(Nat.2, x)
        if Nat.2.gcd(x) = Nat.1 {
            Nat.2.coprime(x)
            coprime_divides_of_divides_mul(Nat.2, x, x)
            Nat.2.divides(x)
        } else {
            Nat.2.divides(x)
        }
    }
}

/// Infinite descent: a nonzero solution of x² = 2y² yields a smaller one.
/// From x² = 2y² both coordinates are even, x = 2a and y = 2b, and then
/// a² = 2b² with a < x.
theorem sq_eq_two_sq_descent(x: Nat, y: Nat) {
    x != Nat.0 and x * x = Nat.2 * (y * y) implies
        exists(a: Nat, b: Nat) {
            a < x and Nat.2 * a = x and a * a = Nat.2 * (b * b)
        }
} by {
    if x != Nat.0 and x * x = Nat.2 * (y * y) {
        // 2 | x·x, hence 2 | x, x = 2a.
        Nat.2.divides(Nat.2 * (y * y))
        Nat.2 * (y * y) = x * x
        Nat.2.divides(x * x)
        two_divides_square_imp_two_divides(x)
        Nat.2.divides(x)
        let (a: Nat) satisfy { Nat.2 * a = x }
        // a is a proper smaller root: a < x.
        mul_to_zero(Nat.2, a)
        if Nat.2 * a != Nat.0 {
            a != Nat.0
        }
        a != Nat.0
        a * Nat.2 = Nat.2 * a
        a * Nat.2 = x
        Nat.1 < Nat.2
        divisor_lt(a, Nat.2, x)
        a < x
        // y·y = 2·a·a from (2a)² = 2y².
        Nat.2 * a = x
        x * x = (Nat.2 * a) * (Nat.2 * a)
        x * x = Nat.2 * (y * y)
        (Nat.2 * a) * (Nat.2 * a) = Nat.2 * (y * y)
        nat_sq_double(a)
        (Nat.2 * a) * (Nat.2 * a) = Nat.4 * (a * a)
        Nat.4 * (a * a) = Nat.2 * (y * y)
        Nat.2 * (Nat.2 * (a * a)) = Nat.2 * (y * y)
        mul_cancel_left(Nat.2, Nat.2 * (a * a), y * y)
        Nat.2 * (a * a) = y * y
        // 2 | y·y, hence 2 | y, y = 2b.
        Nat.2.divides(Nat.2 * (a * a))
        Nat.2 * (a * a) = y * y
        Nat.2.divides(y * y)
        two_divides_square_imp_two_divides(y)
        Nat.2.divides(y)
        let (b: Nat) satisfy { Nat.2 * b = y }
        // a·a = 2·b·b from y = 2b.
        Nat.2 * b = y
        y * y = (Nat.2 * b) * (Nat.2 * b)
        Nat.2 * (a * a) = y * y
        Nat.2 * (a * a) = (Nat.2 * b) * (Nat.2 * b)
        nat_sq_double(b)
        (Nat.2 * b) * (Nat.2 * b) = Nat.4 * (b * b)
        Nat.2 * (a * a) = Nat.4 * (b * b)
        Nat.2 * (a * a) = Nat.2 * (Nat.2 * (b * b))
        mul_cancel_left(Nat.2, a * a, Nat.2 * (b * b))
        a * a = Nat.2 * (b * b)
        // assemble the witness.
        a < x and Nat.2 * a = x and a * a = Nat.2 * (b * b)
        exists(a2: Nat, b2: Nat) {
            a2 < x and Nat.2 * a2 = x and a2 * a2 = Nat.2 * (b2 * b2)
        }
    }
}

/// The infinite descent of sq_eq_two_sq_descent, run by well-founded
/// induction on the first coordinate: x² = 2y² forces x = 0.
theorem sq_eq_two_sq_zero(x: Nat) {
    forall(y: Nat) { x * x = Nat.2 * (y * y) implies x = Nat.0 }
} by {
    let f: Nat -> Bool = function(t: Nat) {
        forall(y: Nat) { t * t = Nat.2 * (y * y) implies t = Nat.0 }
    }
    forall(z: Nat) {
        if forall(k: Nat) { nat_lt_relation(k, z) implies f(k) } {
            forall(y: Nat) {
                if z * z = Nat.2 * (y * y) {
                    if z = Nat.0 {
                        z = Nat.0
                    } else {
                        z != Nat.0 and z * z = Nat.2 * (y * y)
                        sq_eq_two_sq_descent(z, y)
                        let (a: Nat, b: Nat) satisfy {
                            a < z and Nat.2 * a = z and a * a = Nat.2 * (b * b)
                        }
                        nat_lt_relation(a, z) = (a < z)
                        nat_lt_relation(a, z)
                        forall(k: Nat) { nat_lt_relation(k, z) implies f(k) }
                        nat_lt_relation(a, z) implies f(a)
                        f(a)
                        f(a) =
                            forall(w: Nat) { a * a = Nat.2 * (w * w) implies a = Nat.0 }
                        forall(w: Nat) { a * a = Nat.2 * (w * w) implies a = Nat.0 }
                        a * a = Nat.2 * (b * b) implies a = Nat.0
                        a = Nat.0
                        Nat.2 * a = z
                        Nat.2 * Nat.0 = Nat.0
                        z = Nat.0
                        false
                    }
                }
            }
            forall(y: Nat) { z * z = Nat.2 * (y * y) implies z = Nat.0 }
            f(z) = forall(w: Nat) { z * z = Nat.2 * (w * w) implies z = Nat.0 }
            f(z)
        }
    }
    nat_lt_relation_induction_at(f, x)
    f(x)
    f(x) = forall(y: Nat) { x * x = Nat.2 * (y * y) implies x = Nat.0 }
    forall(y: Nat) { x * x = Nat.2 * (y * y) implies x = Nat.0 }
}

/// The Diophantine equation x² = 2y² has no nonzero natural solution.
theorem no_nontrivial_sq_eq_two_sq(x: Nat, y: Nat) {
    x * x = Nat.2 * (y * y) implies x = Nat.0 and y = Nat.0
} by {
    if x * x = Nat.2 * (y * y) {
        sq_eq_two_sq_zero(x)
        forall(w: Nat) { x * x = Nat.2 * (w * w) implies x = Nat.0 }
        x * x = Nat.2 * (y * y) implies x = Nat.0
        x = Nat.0
        x * x = Nat.2 * (y * y)
        Nat.0 * Nat.0 = Nat.0
        Nat.0 = Nat.2 * (y * y)
        Nat.2 * (y * y) = Nat.0
        mul_to_zero(Nat.2, y * y)
        if Nat.2 = Nat.0 {
            false
        }
        y * y = Nat.0
        mul_to_zero(y, y)
        if y = Nat.0 {
        } else {
            false
        }
        y = Nat.0
        x = Nat.0 and y = Nat.0
    }
}

/// The absolute value of the integer two is the natural two.
theorem abs_from_nat_two {
    abs(Int.2) = Nat.2
} by {
    abs_from_nat(Nat.2)
    abs(Int.from_nat(Nat.2)) = Nat.2
    Int.from_nat(Nat.2) = Int.2
    abs(Int.2) = Nat.2
}

/// The Diophantine equation x² = 2y² has no nonzero integer solution either:
/// taking absolute values reduces to the natural-number statement, since
/// |x|² = 2·|y|².
theorem no_nontrivial_int_sq_eq_two_sq(x: Int, y: Int) {
    x * x = Int.2 * (y * y) implies x = Int.0 and y = Int.0
} by {
    if x * x = Int.2 * (y * y) {
        abs_mul(x, x)
        abs(x * x) = abs(x) * abs(x)
        abs_mul(Int.2, y * y)
        abs(Int.2 * (y * y)) = abs(Int.2) * abs(y * y)
        abs_mul(y, y)
        abs(y * y) = abs(y) * abs(y)
        abs(Int.2 * (y * y)) = abs(Int.2) * (abs(y) * abs(y))
        abs_from_nat_two
        abs(Int.2) = Nat.2
        abs(Int.2 * (y * y)) = Nat.2 * (abs(y) * abs(y))
        abs(x * x) = abs(Int.2 * (y * y))
        abs(x) * abs(x) = Nat.2 * (abs(y) * abs(y))
        no_nontrivial_sq_eq_two_sq(abs(x), abs(y))
        abs(x) = Nat.0 and abs(y) = Nat.0
        abs_zero_imp_zero(x)
        abs(x) = Nat.0 implies x = Int.0
        x = Int.0
        abs_zero_imp_zero(y)
        abs(y) = Nat.0 implies y = Int.0
        y = Int.0
        x = Int.0 and y = Int.0
    }
}

// ============================================================================
// Section 4: Fermat's Last Theorem for the exponent three (statement)
// ============================================================================

// Fermat's Last Theorem for the exponent three: the equation x³ + y³ = z³
// has no solution in nonzero natural numbers.  This is the classical
// n = 3 case, whose standard proof runs through the arithmetic of the
// Eisenstein integers Z[ω] (unique factorization, and the descent showing
// that x³ + y³ = z³ would force a smaller solution).  The library does not
// yet formalize the Eisenstein integers, so the statement is recorded here
// without proof, mirroring theorems1000/theorem_fermat_last.ac.
//
// theorem fermat_last_exponent_three {
//     forall(x: Nat, y: Nat, z: Nat) {
//         x != Nat.0 and y != Nat.0 and z != Nat.0
//             implies not (x.pow(Nat.3) + y.pow(Nat.3) = z.pow(Nat.3))
//     }
// }

// ============================================================================
// Section 5: Pell's equation x² - 2y² = 1
// ============================================================================

/// The x-component of the squared Pell solution at the natural level:
/// a² + 2b².
define pell_two_nat_next_x(a: Nat, b: Nat) -> Nat {
    a * a + Nat.2 * (b * b)
}

/// The y-component of the squared Pell solution at the natural level: 2ab.
define pell_two_nat_next_y(a: Nat, b: Nat) -> Nat {
    Nat.2 * a * b
}

/// The natural iteration of the Pell step preserves solutions: if
/// (a, b) solves x² - 2y² = 1, so does (a² + 2b², 2ab).
theorem pell_two_nat_step_solution(a: Nat, b: Nat) {
    is_pell_solution(Int.2, Int.from_nat(a), Int.from_nat(b)) implies
        is_pell_solution(Int.2,
            Int.from_nat(pell_two_nat_next_x(a, b)),
            Int.from_nat(pell_two_nat_next_y(a, b)))
} by {
    if is_pell_solution(Int.2, Int.from_nat(a), Int.from_nat(b)) {
        pell_solution_square(Int.2, Int.from_nat(a), Int.from_nat(b))
        is_pell_solution(Int.2,
            pell_next_x(Int.2, Int.from_nat(a), Int.from_nat(b)),
            pell_next_y(Int.2, Int.from_nat(a), Int.from_nat(b)))
        mul_from_nat(a, a)
        Int.from_nat(a) * Int.from_nat(a) = Int.from_nat(a * a)
        mul_from_nat(b, b)
        Int.from_nat(b) * Int.from_nat(b) = Int.from_nat(b * b)
        mul_from_nat(Nat.2, b * b)
        Int.from_nat(Nat.2) * Int.from_nat(b * b) = Int.from_nat(Nat.2 * (b * b))
        Int.from_nat(Nat.2) = Int.2
        Int.2 * Int.from_nat(b * b) = Int.from_nat(Nat.2 * (b * b))
        add_from_nat(a * a, Nat.2 * (b * b))
        Int.from_nat(a * a) + Int.from_nat(Nat.2 * (b * b)) =
            Int.from_nat(a * a + Nat.2 * (b * b))
        pell_next_x(Int.2, Int.from_nat(a), Int.from_nat(b)) =
            Int.from_nat(pell_two_nat_next_x(a, b))
        mul_from_nat(Nat.2 * a, b)
        Int.from_nat(Nat.2 * a) * Int.from_nat(b) = Int.from_nat((Nat.2 * a) * b)
        mul_from_nat(Nat.2, a)
        Int.from_nat(Nat.2) * Int.from_nat(a) = Int.from_nat(Nat.2 * a)
        Int.from_nat(Nat.2) = Int.2
        Int.2 * Int.from_nat(a) = Int.from_nat(Nat.2 * a)
        Int.2 * Int.from_nat(a) * Int.from_nat(b) = Int.from_nat(Nat.2 * a * b)
        pell_next_y(Int.2, Int.from_nat(a), Int.from_nat(b)) =
            Int.from_nat(pell_two_nat_next_y(a, b))
        is_pell_solution(Int.2,
            Int.from_nat(pell_two_nat_next_x(a, b)),
            Int.from_nat(pell_two_nat_next_y(a, b)))
    }
}

/// The squared Pell step keeps the coordinates positive.
theorem pell_two_nat_step_pos(a: Nat, b: Nat) {
    Nat.1 <= a and Nat.1 <= b implies
        Nat.1 <= pell_two_nat_next_x(a, b) and Nat.1 <= pell_two_nat_next_y(a, b)
} by {
    if Nat.1 <= a and Nat.1 <= b {
        // 1 <= a·a + 2·b·b
        a != Nat.0
        lte_mul(a, a)
        a <= a * a
        lte_trans(Nat.1, a, a * a)
        Nat.1 <= a * a
        Nat.0 <= Nat.2 * (b * b)
        lte_add_left(a * a, Nat.0, Nat.2 * (b * b))
        a * a + Nat.0 <= a * a + Nat.2 * (b * b)
        a * a + Nat.0 = a * a
        a * a <= a * a + Nat.2 * (b * b)
        lte_trans(Nat.1, a * a, a * a + Nat.2 * (b * b))
        Nat.1 <= a * a + Nat.2 * (b * b)
        Nat.1 <= pell_two_nat_next_x(a, b)
        // 1 <= 2ab
        Nat.2 * a != Nat.0
        lte_mul(b, Nat.2 * a)
        b <= b * (Nat.2 * a)
        b * (Nat.2 * a) = Nat.2 * a * b
        b <= Nat.2 * a * b
        lte_trans(Nat.1, b, Nat.2 * a * b)
        Nat.1 <= Nat.2 * a * b
        Nat.1 <= pell_two_nat_next_y(a, b)
        Nat.1 <= pell_two_nat_next_x(a, b) and Nat.1 <= pell_two_nat_next_y(a, b)
    }
}

/// The squared Pell step strictly grows the y-coordinate: 2ab >= b + 1.
theorem pell_two_nat_step_grows(a: Nat, b: Nat) {
    Nat.1 <= a and Nat.1 <= b implies b + Nat.1 <= pell_two_nat_next_y(a, b)
} by {
    if Nat.1 <= a and Nat.1 <= b {
        // b + 1 <= b + b = 2b <= 2ab.
        lte_add_left(b, Nat.1, b)
        b + Nat.1 <= b + b
        b + b = Nat.2 * b
        b + Nat.1 <= Nat.2 * b
        lte_mul_both(b, Nat.2, Nat.2 * a)
        Nat.1 <= a
        lte_mul_both(Nat.2, Nat.1, a)
        Nat.2 * Nat.1 <= Nat.2 * a
        Nat.2 * Nat.1 = Nat.2
        Nat.2 <= Nat.2 * a
        b * Nat.2 <= b * (Nat.2 * a)
        b * Nat.2 = Nat.2 * b
        Nat.2 * b <= b * (Nat.2 * a)
        b * (Nat.2 * a) = Nat.2 * a * b
        Nat.2 * b <= Nat.2 * a * b
        lte_trans(b + Nat.1, Nat.2 * b, Nat.2 * a * b)
        b + Nat.1 <= Nat.2 * a * b
        b + Nat.1 <= pell_two_nat_next_y(a, b)
    }
}

/// Predicate of the unboundedness induction: beyond every bound there is a
/// solution (a, b) of x² - 2y² = 1 with a >= 1, b >= 1, and m <= b.
define pell_two_unbounded_pred(m: Nat) -> Bool {
    exists(a: Nat, b: Nat) {
        is_pell_solution(Int.2, Int.from_nat(a), Int.from_nat(b)) and
        Nat.1 <= a and Nat.1 <= b and m <= b
    }
}

/// The base of the unboundedness induction: (3, 2) solves x² - 2y² = 1.
theorem pell_two_unbounded_base {
    pell_two_unbounded_pred(Nat.0)
} by {
    pell_two_smallest
    is_pell_solution(Int.2, Int.3, Int.2)
    Int.from_nat(Nat.3) = Int.3
    Int.from_nat(Nat.2) = Int.2
    is_pell_solution(Int.2, Int.from_nat(Nat.3), Int.from_nat(Nat.2))
    Nat.1 <= Nat.3
    Nat.1 <= Nat.2
    Nat.0 <= Nat.2
    is_pell_solution(Int.2, Int.from_nat(Nat.3), Int.from_nat(Nat.2)) and
        Nat.1 <= Nat.3 and Nat.1 <= Nat.2 and Nat.0 <= Nat.2
    pell_two_unbounded_pred(Nat.0) =
        exists(a: Nat, b: Nat) {
            is_pell_solution(Int.2, Int.from_nat(a), Int.from_nat(b)) and
            Nat.1 <= a and Nat.1 <= b and Nat.0 <= b
        }
    pell_two_unbounded_pred(Nat.0)
}

/// The step of the unboundedness induction: from a solution with y >= m, the
/// squared step gives one with y >= m + 1.
theorem pell_two_unbounded_step(m: Nat) {
    pell_two_unbounded_pred(m) implies pell_two_unbounded_pred(m.suc)
} by {
    if pell_two_unbounded_pred(m) {
        pell_two_unbounded_pred(m) =
            exists(a: Nat, b: Nat) {
                is_pell_solution(Int.2, Int.from_nat(a), Int.from_nat(b)) and
                Nat.1 <= a and Nat.1 <= b and m <= b
            }
        let (a: Nat, b: Nat) satisfy {
            is_pell_solution(Int.2, Int.from_nat(a), Int.from_nat(b)) and
            Nat.1 <= a and Nat.1 <= b and m <= b
        }
        pell_two_nat_step_solution(a, b)
        is_pell_solution(Int.2,
            Int.from_nat(pell_two_nat_next_x(a, b)),
            Int.from_nat(pell_two_nat_next_y(a, b)))
        pell_two_nat_step_pos(a, b)
        Nat.1 <= pell_two_nat_next_x(a, b) and Nat.1 <= pell_two_nat_next_y(a, b)
        pell_two_nat_step_grows(a, b)
        b + Nat.1 <= pell_two_nat_next_y(a, b)
        m <= b
        lte_add_left(Nat.1, m, b)
        m + Nat.1 <= b + Nat.1
        lte_trans(m + Nat.1, b + Nat.1, pell_two_nat_next_y(a, b))
        m + Nat.1 <= pell_two_nat_next_y(a, b)
        m.suc = m + Nat.1
        m.suc <= pell_two_nat_next_y(a, b)
        is_pell_solution(Int.2,
            Int.from_nat(pell_two_nat_next_x(a, b)),
            Int.from_nat(pell_two_nat_next_y(a, b))) and
            Nat.1 <= pell_two_nat_next_x(a, b) and
            Nat.1 <= pell_two_nat_next_y(a, b) and
            m.suc <= pell_two_nat_next_y(a, b)
        pell_two_unbounded_pred(m.suc) =
            exists(a2: Nat, b2: Nat) {
                is_pell_solution(Int.2, Int.from_nat(a2), Int.from_nat(b2)) and
                Nat.1 <= a2 and Nat.1 <= b2 and m.suc <= b2
            }
        pell_two_unbounded_pred(m.suc)
    }
}

/// Pell's equation x² - 2y² = 1 has infinitely many solutions: beyond every
/// bound n there is a solution with y-coordinate at least n.
theorem pell_two_solutions_unbounded(n: Nat) {
    exists(a: Nat, b: Nat) {
        is_pell_solution(Int.2, Int.from_nat(a), Int.from_nat(b)) and n <= b
    }
} by {
    define p(m: Nat) -> Bool {
        exists(a: Nat, b: Nat) {
            is_pell_solution(Int.2, Int.from_nat(a), Int.from_nat(b)) and
            Nat.1 <= a and Nat.1 <= b and m <= b
        }
    }
    forall(y: Nat) {
        pell_two_unbounded_pred(y) = p(y)
        p(y) = pell_two_unbounded_pred(y)
    }
    pell_two_unbounded_base
    pell_two_unbounded_pred(Nat.0)
    p(Nat.0)
    forall(m: Nat) {
        if p(m) {
            pell_two_unbounded_pred(m)
            pell_two_unbounded_step(m)
            pell_two_unbounded_pred(m.suc)
            p(m.suc)
        }
    }
    forall(m: Nat) {
        p(m) implies p(m.suc)
    }
    p(Nat.0) and forall(m: Nat) { p(m) implies p(m.suc) }
    alt_induction(p)
    forall(m: Nat) {
        p(m)
    }
    p(n)
    pell_two_unbounded_pred(n)
    pell_two_unbounded_pred(n) =
        exists(a: Nat, b: Nat) {
            is_pell_solution(Int.2, Int.from_nat(a), Int.from_nat(b)) and
            Nat.1 <= a and Nat.1 <= b and n <= b
        }
    exists(a: Nat, b: Nat) {
        is_pell_solution(Int.2, Int.from_nat(a), Int.from_nat(b)) and n <= b
    }
}

// Pell's equation x² - D·y² = 1 for a general nonsquare D.  The composition
// law pell_solution_square in pell.ac holds for every d, and iterating it
// from the fundamental solution gives infinitely many solutions whenever the
// fundamental solution is known.  The library proves the fundamental solution
// only for d = 2 (pell.ac, Section 2); the general existence of the
// fundamental solution from the continued fraction of √D would require the
// convergence of continued fractions to √D as a real number, which the
// library does not yet formalize (see the comment at the end of pell.ac).
//
// theorem pell_infinitely_many_solutions(d: Nat) {
//     forall(n: Nat) {
//         exists(a: Nat, b: Nat) {
//             is_pell_solution(Int.from_nat(d), Int.from_nat(a), Int.from_nat(b)) and
//             n <= b
//         }
//     }
// }
