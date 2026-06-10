from nat import Nat, carry_count_fuel, digit_sum, addition_carry_count,
    carry_count_fuel_eq_addition_carry_count_of_sum_le_fuel,
    double_addition_carry_count, double_addition_carry_count_eq,
    is_addition_carry_count, is_addition_carry_count_eq_addition_carry_count,
    lte_ref
from combinatorics import binom, choose_symm_add_form
from number_theory.factorisation import count_prime_factor
from number_theory.kummer import legendre_digit_sum, kummer_digit_sum_of_legendre_digit_sum
from number_theory.falling_product import central_binom, central_binom_eq,
    central_binom_two_adic_valuation, nat_two_prime
numerals Nat

/// The p-adic valuation of `binom(a+b,a)` satisfies the digit-sum equation
/// characterizing the carry count for adding `a` and `b` in base `p`.
theorem binom_valuation_is_addition_carry_count(p: Nat, a: Nat, b: Nat) {
    p.is_prime implies
        is_addition_carry_count(p, a, b,
            count_prime_factor(p, (a + b).binom(a)))
} by {
    if p.is_prime {
        legendre_digit_sum(p, a)
        legendre_digit_sum(p, b)
        legendre_digit_sum(p, a + b)
        kummer_digit_sum_of_legendre_digit_sum(p, a, b)
        p * count_prime_factor(p, (a + b).binom(a)) + digit_sum(p, a + b) =
            count_prime_factor(p, (a + b).binom(a)) +
                digit_sum(p, a) + digit_sum(p, b)
        is_addition_carry_count(p, a, b,
            count_prime_factor(p, (a + b).binom(a)))
    }
}

/// The p-adic valuation of the central binomial coefficient satisfies the
/// carry-count digit-sum equation for doubling `n` in base `p`.
theorem central_binom_valuation_is_addition_carry_count(p: Nat, n: Nat) {
    p.is_prime implies
        is_addition_carry_count(p, n, n,
            count_prime_factor(p, central_binom(n)))
} by {
    if p.is_prime {
        binom_valuation_is_addition_carry_count(p, n, n)
        is_addition_carry_count(p, n, n,
            count_prime_factor(p, (n + n).binom(n)))
        central_binom_eq(n)
        central_binom(n) = (n + n).binom(n)
        count_prime_factor(p, central_binom(n)) =
            count_prime_factor(p, (n + n).binom(n))
        is_addition_carry_count(p, n, n,
            count_prime_factor(p, central_binom(n)))
    }
}

/// The p-adic valuation of `binom(a+b,a)` is the recursive carry count for
/// adding `a` and `b` in base `p`.
theorem binom_valuation_eq_addition_carry_count(p: Nat, a: Nat, b: Nat) {
    p.is_prime implies
        count_prime_factor(p, (a + b).binom(a)) =
            addition_carry_count(p, a, b)
} by {
    if p.is_prime {
        Nat.1 < p
        binom_valuation_is_addition_carry_count(p, a, b)
        is_addition_carry_count(p, a, b,
            count_prime_factor(p, (a + b).binom(a)))
        is_addition_carry_count_eq_addition_carry_count(
            p, a, b, count_prime_factor(p, (a + b).binom(a)))
        count_prime_factor(p, (a + b).binom(a)) =
            addition_carry_count(p, a, b)
    }
}

/// The recursive carry count for adding `a` and `b` in base `p` is the
/// p-adic valuation of `binom(a+b,a)`.
theorem addition_carry_count_eq_binom_valuation(p: Nat, a: Nat, b: Nat) {
    p.is_prime implies
        addition_carry_count(p, a, b) =
            count_prime_factor(p, (a + b).binom(a))
} by {
    if p.is_prime {
        binom_valuation_eq_addition_carry_count(p, a, b)
        count_prime_factor(p, (a + b).binom(a)) =
            addition_carry_count(p, a, b)
        addition_carry_count(p, a, b) =
            count_prime_factor(p, (a + b).binom(a))
    }
}

/// The p-adic valuation of `binom(a+b,b)` is the recursive carry count for
/// adding `a` and `b` in base `p`.
theorem symmetric_binom_valuation_eq_addition_carry_count(
    p: Nat, a: Nat, b: Nat
) {
    p.is_prime implies
        count_prime_factor(p, (a + b).binom(b)) =
            addition_carry_count(p, a, b)
} by {
    if p.is_prime {
        choose_symm_add_form(a, b)
        (a + b).binom(a) = (a + b).binom(b)
        count_prime_factor(p, (a + b).binom(a)) =
            count_prime_factor(p, (a + b).binom(b))
        binom_valuation_eq_addition_carry_count(p, a, b)
        count_prime_factor(p, (a + b).binom(a)) =
            addition_carry_count(p, a, b)
        count_prime_factor(p, (a + b).binom(b)) =
            addition_carry_count(p, a, b)
    }
}

/// The recursive carry count for adding `a` and `b` in base `p` is the
/// p-adic valuation of `binom(a+b,b)`.
theorem addition_carry_count_eq_symmetric_binom_valuation(
    p: Nat, a: Nat, b: Nat
) {
    p.is_prime implies
        addition_carry_count(p, a, b) =
            count_prime_factor(p, (a + b).binom(b))
} by {
    if p.is_prime {
        symmetric_binom_valuation_eq_addition_carry_count(p, a, b)
        count_prime_factor(p, (a + b).binom(b)) =
            addition_carry_count(p, a, b)
        addition_carry_count(p, a, b) =
            count_prime_factor(p, (a + b).binom(b))
    }
}

/// Any sufficiently large zero-incoming fuel computes the p-adic valuation of
/// `binom(a+b,a)`.
theorem carry_count_fuel_eq_binom_valuation_of_sum_le_fuel(
    p: Nat, a: Nat, b: Nat, fuel: Nat
) {
    p.is_prime and a + b <= fuel implies
        carry_count_fuel(p, a, b, Nat.0, fuel) =
            count_prime_factor(p, (a + b).binom(a))
} by {
    if p.is_prime and a + b <= fuel {
        Nat.1 < p
        carry_count_fuel_eq_addition_carry_count_of_sum_le_fuel(
            p, a, b, fuel)
        carry_count_fuel(p, a, b, Nat.0, fuel) =
            addition_carry_count(p, a, b)
        addition_carry_count_eq_binom_valuation(p, a, b)
        addition_carry_count(p, a, b) =
            count_prime_factor(p, (a + b).binom(a))
        carry_count_fuel(p, a, b, Nat.0, fuel) =
            count_prime_factor(p, (a + b).binom(a))
    }
}

/// The p-adic valuation of `binom(a+b,a)` is computed by any sufficiently
/// large zero-incoming fuel.
theorem binom_valuation_eq_carry_count_fuel_of_sum_le_fuel(
    p: Nat, a: Nat, b: Nat, fuel: Nat
) {
    p.is_prime and a + b <= fuel implies
        count_prime_factor(p, (a + b).binom(a)) =
            carry_count_fuel(p, a, b, Nat.0, fuel)
} by {
    if p.is_prime and a + b <= fuel {
        carry_count_fuel_eq_binom_valuation_of_sum_le_fuel(p, a, b, fuel)
        carry_count_fuel(p, a, b, Nat.0, fuel) =
            count_prime_factor(p, (a + b).binom(a))
        count_prime_factor(p, (a + b).binom(a)) =
            carry_count_fuel(p, a, b, Nat.0, fuel)
    }
}

/// The canonical fuel `a+b` computes the p-adic valuation of
/// `binom(a+b,a)`.
theorem carry_count_fuel_at_sum_eq_binom_valuation(
    p: Nat, a: Nat, b: Nat
) {
    p.is_prime implies
        carry_count_fuel(p, a, b, Nat.0, a + b) =
            count_prime_factor(p, (a + b).binom(a))
} by {
    if p.is_prime {
        lte_ref(a + b)
        a + b <= a + b
        carry_count_fuel_eq_binom_valuation_of_sum_le_fuel(p, a, b, a + b)
        carry_count_fuel(p, a, b, Nat.0, a + b) =
            count_prime_factor(p, (a + b).binom(a))
    }
}

/// The p-adic valuation of `binom(a+b,a)` is computed by the canonical fuel
/// `a+b`.
theorem binom_valuation_eq_carry_count_fuel_at_sum(
    p: Nat, a: Nat, b: Nat
) {
    p.is_prime implies
        count_prime_factor(p, (a + b).binom(a)) =
            carry_count_fuel(p, a, b, Nat.0, a + b)
} by {
    if p.is_prime {
        carry_count_fuel_at_sum_eq_binom_valuation(p, a, b)
        carry_count_fuel(p, a, b, Nat.0, a + b) =
            count_prime_factor(p, (a + b).binom(a))
        count_prime_factor(p, (a + b).binom(a)) =
            carry_count_fuel(p, a, b, Nat.0, a + b)
    }
}

/// The canonical fuel `a+b` computes the p-adic valuation of
/// `binom(a+b,b)`.
theorem carry_count_fuel_at_sum_eq_symmetric_binom_valuation(
    p: Nat, a: Nat, b: Nat
) {
    p.is_prime implies
        carry_count_fuel(p, a, b, Nat.0, a + b) =
            count_prime_factor(p, (a + b).binom(b))
} by {
    if p.is_prime {
        carry_count_fuel_at_sum_eq_binom_valuation(p, a, b)
        carry_count_fuel(p, a, b, Nat.0, a + b) =
            count_prime_factor(p, (a + b).binom(a))
        choose_symm_add_form(a, b)
        (a + b).binom(a) = (a + b).binom(b)
        count_prime_factor(p, (a + b).binom(a)) =
            count_prime_factor(p, (a + b).binom(b))
        carry_count_fuel(p, a, b, Nat.0, a + b) =
            count_prime_factor(p, (a + b).binom(b))
    }
}

/// The p-adic valuation of `binom(a+b,b)` is computed by the canonical fuel
/// `a+b`.
theorem symmetric_binom_valuation_eq_carry_count_fuel_at_sum(
    p: Nat, a: Nat, b: Nat
) {
    p.is_prime implies
        count_prime_factor(p, (a + b).binom(b)) =
            carry_count_fuel(p, a, b, Nat.0, a + b)
} by {
    if p.is_prime {
        carry_count_fuel_at_sum_eq_symmetric_binom_valuation(p, a, b)
        carry_count_fuel(p, a, b, Nat.0, a + b) =
            count_prime_factor(p, (a + b).binom(b))
        count_prime_factor(p, (a + b).binom(b)) =
            carry_count_fuel(p, a, b, Nat.0, a + b)
    }
}

/// The p-adic valuation of the central binomial coefficient is the recursive
/// carry count for adding `n` to itself in base `p`.
theorem central_binom_valuation_eq_addition_carry_count(p: Nat, n: Nat) {
    p.is_prime implies
        count_prime_factor(p, central_binom(n)) =
            addition_carry_count(p, n, n)
} by {
    if p.is_prime {
        central_binom_valuation_is_addition_carry_count(p, n)
        is_addition_carry_count(p, n, n,
            count_prime_factor(p, central_binom(n)))
        Nat.1 < p
        is_addition_carry_count_eq_addition_carry_count(
            p, n, n, count_prime_factor(p, central_binom(n)))
        count_prime_factor(p, central_binom(n)) =
            addition_carry_count(p, n, n)
    }
}

/// The recursive carry count for adding `n` to itself in base `p` is the
/// p-adic valuation of the central binomial coefficient.
theorem addition_carry_count_eq_central_binom_valuation(p: Nat, n: Nat) {
    p.is_prime implies
        addition_carry_count(p, n, n) =
            count_prime_factor(p, central_binom(n))
} by {
    if p.is_prime {
        central_binom_valuation_eq_addition_carry_count(p, n)
        count_prime_factor(p, central_binom(n)) =
            addition_carry_count(p, n, n)
        addition_carry_count(p, n, n) =
            count_prime_factor(p, central_binom(n))
    }
}

/// The p-adic valuation of the central binomial coefficient is the
/// doubled-addend recursive carry count in base `p`.
theorem central_binom_valuation_eq_double_addition_carry_count(p: Nat, n: Nat) {
    p.is_prime implies
        count_prime_factor(p, central_binom(n)) =
            double_addition_carry_count(p, n)
} by {
    if p.is_prime {
        central_binom_valuation_eq_addition_carry_count(p, n)
        count_prime_factor(p, central_binom(n)) =
            addition_carry_count(p, n, n)
        double_addition_carry_count_eq(p, n)
        double_addition_carry_count(p, n) = addition_carry_count(p, n, n)
        count_prime_factor(p, central_binom(n)) =
            double_addition_carry_count(p, n)
    }
}

/// The doubled-addend recursive carry count in base `p` is the p-adic
/// valuation of the central binomial coefficient.
theorem double_addition_carry_count_eq_central_binom_valuation(p: Nat, n: Nat) {
    p.is_prime implies
        double_addition_carry_count(p, n) =
            count_prime_factor(p, central_binom(n))
} by {
    if p.is_prime {
        central_binom_valuation_eq_double_addition_carry_count(p, n)
        count_prime_factor(p, central_binom(n)) =
            double_addition_carry_count(p, n)
        double_addition_carry_count(p, n) =
            count_prime_factor(p, central_binom(n))
    }
}

/// Any sufficiently large fuel for doubling computes the p-adic valuation of
/// the central binomial coefficient.
theorem carry_count_fuel_eq_central_binom_valuation_of_double_le_fuel(
    p: Nat, n: Nat, fuel: Nat
) {
    p.is_prime and n + n <= fuel implies
        carry_count_fuel(p, n, n, Nat.0, fuel) =
            count_prime_factor(p, central_binom(n))
} by {
    if p.is_prime and n + n <= fuel {
        Nat.1 < p
        carry_count_fuel_eq_addition_carry_count_of_sum_le_fuel(
            p, n, n, fuel)
        carry_count_fuel(p, n, n, Nat.0, fuel) =
            addition_carry_count(p, n, n)
        addition_carry_count_eq_central_binom_valuation(p, n)
        addition_carry_count(p, n, n) =
            count_prime_factor(p, central_binom(n))
        carry_count_fuel(p, n, n, Nat.0, fuel) =
            count_prime_factor(p, central_binom(n))
    }
}

/// The p-adic valuation of the central binomial coefficient is computed by
/// any sufficiently large fuel for doubling.
theorem central_binom_valuation_eq_carry_count_fuel_of_double_le_fuel(
    p: Nat, n: Nat, fuel: Nat
) {
    p.is_prime and n + n <= fuel implies
        count_prime_factor(p, central_binom(n)) =
            carry_count_fuel(p, n, n, Nat.0, fuel)
} by {
    if p.is_prime and n + n <= fuel {
        carry_count_fuel_eq_central_binom_valuation_of_double_le_fuel(
            p, n, fuel)
        carry_count_fuel(p, n, n, Nat.0, fuel) =
            count_prime_factor(p, central_binom(n))
        count_prime_factor(p, central_binom(n)) =
            carry_count_fuel(p, n, n, Nat.0, fuel)
    }
}

/// The canonical doubling fuel `n+n` computes the p-adic valuation of the
/// central binomial coefficient.
theorem carry_count_fuel_at_double_sum_eq_central_binom_valuation(
    p: Nat, n: Nat
) {
    p.is_prime implies
        carry_count_fuel(p, n, n, Nat.0, n + n) =
            count_prime_factor(p, central_binom(n))
} by {
    if p.is_prime {
        lte_ref(n + n)
        n + n <= n + n
        carry_count_fuel_eq_central_binom_valuation_of_double_le_fuel(
            p, n, n + n)
        carry_count_fuel(p, n, n, Nat.0, n + n) =
            count_prime_factor(p, central_binom(n))
    }
}

/// The p-adic valuation of the central binomial coefficient is computed by
/// the canonical doubling fuel `n+n`.
theorem central_binom_valuation_eq_carry_count_fuel_at_double_sum(
    p: Nat, n: Nat
) {
    p.is_prime implies
        count_prime_factor(p, central_binom(n)) =
            carry_count_fuel(p, n, n, Nat.0, n + n)
} by {
    if p.is_prime {
        carry_count_fuel_at_double_sum_eq_central_binom_valuation(p, n)
        carry_count_fuel(p, n, n, Nat.0, n + n) =
            count_prime_factor(p, central_binom(n))
        count_prime_factor(p, central_binom(n)) =
            carry_count_fuel(p, n, n, Nat.0, n + n)
    }
}

/// The binary doubled-addend carry count is the binary digit sum.
theorem double_addition_carry_count_two_eq_digit_sum(n: Nat) {
    double_addition_carry_count(Nat.2, n) = digit_sum(Nat.2, n)
} by {
    nat_two_prime
    double_addition_carry_count_eq_central_binom_valuation(Nat.2, n)
    double_addition_carry_count(Nat.2, n) =
        count_prime_factor(Nat.2, central_binom(n))
    central_binom_two_adic_valuation(n)
    count_prime_factor(Nat.2, central_binom(n)) = digit_sum(Nat.2, n)
    double_addition_carry_count(Nat.2, n) = digit_sum(Nat.2, n)
}

/// The binary digit sum is the binary doubled-addend carry count.
theorem digit_sum_eq_double_addition_carry_count_two(n: Nat) {
    digit_sum(Nat.2, n) = double_addition_carry_count(Nat.2, n)
} by {
    double_addition_carry_count_two_eq_digit_sum(n)
    double_addition_carry_count(Nat.2, n) = digit_sum(Nat.2, n)
    digit_sum(Nat.2, n) = double_addition_carry_count(Nat.2, n)
}

/// The binary recursive carry count for adding `n` to itself is the binary
/// digit sum.
theorem addition_carry_count_two_double_eq_digit_sum(n: Nat) {
    addition_carry_count(Nat.2, n, n) = digit_sum(Nat.2, n)
} by {
    double_addition_carry_count_eq(Nat.2, n)
    double_addition_carry_count(Nat.2, n) = addition_carry_count(Nat.2, n, n)
    double_addition_carry_count_two_eq_digit_sum(n)
    double_addition_carry_count(Nat.2, n) = digit_sum(Nat.2, n)
    addition_carry_count(Nat.2, n, n) = digit_sum(Nat.2, n)
}

/// Any sufficiently large binary fuel for doubling computes the binary digit
/// sum.
theorem carry_count_fuel_two_double_eq_digit_sum_of_double_le_fuel(
    n: Nat, fuel: Nat
) {
    n + n <= fuel implies
        carry_count_fuel(Nat.2, n, n, Nat.0, fuel) = digit_sum(Nat.2, n)
} by {
    if n + n <= fuel {
        nat_two_prime
        carry_count_fuel_eq_central_binom_valuation_of_double_le_fuel(
            Nat.2, n, fuel)
        carry_count_fuel(Nat.2, n, n, Nat.0, fuel) =
            count_prime_factor(Nat.2, central_binom(n))
        central_binom_two_adic_valuation(n)
        count_prime_factor(Nat.2, central_binom(n)) = digit_sum(Nat.2, n)
        carry_count_fuel(Nat.2, n, n, Nat.0, fuel) = digit_sum(Nat.2, n)
    }
}

/// The binary digit sum is computed by any sufficiently large binary fuel for
/// doubling.
theorem digit_sum_eq_carry_count_fuel_two_double_of_double_le_fuel(
    n: Nat, fuel: Nat
) {
    n + n <= fuel implies
        digit_sum(Nat.2, n) = carry_count_fuel(Nat.2, n, n, Nat.0, fuel)
} by {
    if n + n <= fuel {
        carry_count_fuel_two_double_eq_digit_sum_of_double_le_fuel(n, fuel)
        carry_count_fuel(Nat.2, n, n, Nat.0, fuel) = digit_sum(Nat.2, n)
        digit_sum(Nat.2, n) = carry_count_fuel(Nat.2, n, n, Nat.0, fuel)
    }
}

/// The canonical binary doubling fuel computes the binary digit sum.
theorem carry_count_fuel_two_double_at_double_sum_eq_digit_sum(n: Nat) {
    carry_count_fuel(Nat.2, n, n, Nat.0, n + n) = digit_sum(Nat.2, n)
} by {
    lte_ref(n + n)
    n + n <= n + n
    carry_count_fuel_two_double_eq_digit_sum_of_double_le_fuel(n, n + n)
    carry_count_fuel(Nat.2, n, n, Nat.0, n + n) = digit_sum(Nat.2, n)
}

/// The binary digit sum is computed by the canonical binary doubling fuel.
theorem digit_sum_eq_carry_count_fuel_two_double_at_double_sum(n: Nat) {
    digit_sum(Nat.2, n) = carry_count_fuel(Nat.2, n, n, Nat.0, n + n)
} by {
    carry_count_fuel_two_double_at_double_sum_eq_digit_sum(n)
    carry_count_fuel(Nat.2, n, n, Nat.0, n + n) = digit_sum(Nat.2, n)
    digit_sum(Nat.2, n) = carry_count_fuel(Nat.2, n, n, Nat.0, n + n)
}
