from number_theory.primitive_root_discrete_log import Nat, discrete_log_mod_with_order,
    discrete_log_mod_with_order_congr, unit_discrete_log_mod_with_order_congr,
    discrete_log_mod_with_order_mul
from number_theory.primitive_root import is_power_of_mod, powers_cover_units_mod,
    powers_cover_units_mod_apply, even_power_quadratic_residue_mod,
    is_order_double_unit_generator_mod,
    order_double_half_power_imp_even_exponent
from number_theory.multiplicative_order import is_multiplicative_order_mod
from number_theory.quadratic_residue import is_quadratic_residue_mod,
    is_unit_quadratic_residue_mod, quadratic_residue_coprime_is_unit,
    unit_quadratic_residue_coprime, congr_mod_preserves_coprime,
    euler_criterion_unit_quadratic_residue_forward
from number_theory.congruence import congr_mod_symm, congr_mod_trans, congr_mod_pow
from number_theory.coprime import coprime_mul
numerals Nat

/// A represented residue with even explicit-order discrete logarithm is a
/// quadratic residue.
theorem discrete_log_even_imp_quadratic_residue_mod(
    g: Nat, n: Nat, ord: Nat, a: Nat
) {
    is_multiplicative_order_mod(g, n, ord) and is_power_of_mod(a, g, n)
        and Nat.2.divides(discrete_log_mod_with_order(g, n, ord, a))
        implies is_quadratic_residue_mod(a, n)
} by {
    if is_multiplicative_order_mod(g, n, ord) and is_power_of_mod(a, g, n)
        and Nat.2.divides(discrete_log_mod_with_order(g, n, ord, a)) {
        discrete_log_mod_with_order_congr(g, n, ord, a)
        a.congr_mod(g.pow(discrete_log_mod_with_order(g, n, ord, a)), n)
        even_power_quadratic_residue_mod(
            g, discrete_log_mod_with_order(g, n, ord, a), a, n)
        is_quadratic_residue_mod(a, n)
    }
}

/// A unit square modulo an odd prime has even discrete logarithm with respect
/// to an order-`2*h` unit generator.
theorem unit_quadratic_residue_imp_discrete_log_even(
    p: Nat, h: Nat, g: Nat, a: Nat
) {
    p.is_prime and p = Nat.2 * h + Nat.1
        and is_order_double_unit_generator_mod(g, p, h)
        and is_unit_quadratic_residue_mod(a, p)
        implies Nat.2.divides(discrete_log_mod_with_order(g, p, Nat.2 * h, a))
} by {
    if p.is_prime and p = Nat.2 * h + Nat.1
        and is_order_double_unit_generator_mod(g, p, h)
        and is_unit_quadratic_residue_mod(a, p) {
        is_order_double_unit_generator_mod(g, p, h) =
            (p != Nat.0 and g.coprime(p) and h != Nat.0 and
                is_multiplicative_order_mod(g, p, Nat.2 * h) and
                powers_cover_units_mod(g, p))
        unit_quadratic_residue_coprime(a, p)
        a.coprime(p)
        unit_discrete_log_mod_with_order_congr(g, p, Nat.2 * h, a)
        a.congr_mod(g.pow(discrete_log_mod_with_order(g, p, Nat.2 * h, a)), p)
        congr_mod_pow(
            a, g.pow(discrete_log_mod_with_order(g, p, Nat.2 * h, a)), p, h)
        a.pow(h).congr_mod(
            g.pow(discrete_log_mod_with_order(g, p, Nat.2 * h, a)).pow(h), p)
        euler_criterion_unit_quadratic_residue_forward(p, h, a)
        a.pow(h).congr_mod(Nat.1, p)
        congr_mod_symm(
            a.pow(h),
            g.pow(discrete_log_mod_with_order(g, p, Nat.2 * h, a)).pow(h), p)
        g.pow(discrete_log_mod_with_order(g, p, Nat.2 * h, a)).pow(h).congr_mod(
            a.pow(h), p)
        congr_mod_trans(
            g.pow(discrete_log_mod_with_order(g, p, Nat.2 * h, a)).pow(h),
            a.pow(h), Nat.1, p)
        g.pow(discrete_log_mod_with_order(g, p, Nat.2 * h, a)).pow(h).congr_mod(
            Nat.1, p)
        order_double_half_power_imp_even_exponent(
            p, h, g, discrete_log_mod_with_order(g, p, Nat.2 * h, a))
        Nat.2.divides(discrete_log_mod_with_order(g, p, Nat.2 * h, a))
    }
}

/// For a unit modulo an odd prime, being a square is equivalent to having an
/// even discrete logarithm with respect to an order-`2*h` unit generator.
theorem unit_quadratic_residue_iff_discrete_log_even(
    p: Nat, h: Nat, g: Nat, a: Nat
) {
    p.is_prime and p = Nat.2 * h + Nat.1
        and is_order_double_unit_generator_mod(g, p, h) and a.coprime(p)
        implies (is_unit_quadratic_residue_mod(a, p) =
            Nat.2.divides(discrete_log_mod_with_order(g, p, Nat.2 * h, a)))
} by {
    if p.is_prime and p = Nat.2 * h + Nat.1
        and is_order_double_unit_generator_mod(g, p, h) and a.coprime(p) {
        is_order_double_unit_generator_mod(g, p, h) =
            (p != Nat.0 and g.coprime(p) and h != Nat.0 and
                is_multiplicative_order_mod(g, p, Nat.2 * h) and
                powers_cover_units_mod(g, p))
        powers_cover_units_mod_apply(g, p, a)
        is_power_of_mod(a, g, p)
        if is_unit_quadratic_residue_mod(a, p) {
            unit_quadratic_residue_imp_discrete_log_even(p, h, g, a)
            Nat.2.divides(discrete_log_mod_with_order(g, p, Nat.2 * h, a))
        }
        if Nat.2.divides(discrete_log_mod_with_order(g, p, Nat.2 * h, a)) {
            discrete_log_even_imp_quadratic_residue_mod(g, p, Nat.2 * h, a)
            is_quadratic_residue_mod(a, p)
            quadratic_residue_coprime_is_unit(a, p)
            is_unit_quadratic_residue_mod(a, p)
        }
        (is_unit_quadratic_residue_mod(a, p) =
            Nat.2.divides(discrete_log_mod_with_order(g, p, Nat.2 * h, a))) = true
    }
}

/// A congruent unit target satisfies the same discrete-log parity criterion.
theorem congruent_unit_quadratic_residue_iff_discrete_log_even(
    p: Nat, h: Nat, g: Nat, a: Nat, b: Nat
) {
    p.is_prime and p = Nat.2 * h + Nat.1
        and is_order_double_unit_generator_mod(g, p, h)
        and a.coprime(p) and a.congr_mod(b, p)
        implies (is_unit_quadratic_residue_mod(b, p) =
            Nat.2.divides(discrete_log_mod_with_order(g, p, Nat.2 * h, b)))
} by {
    if p.is_prime and p = Nat.2 * h + Nat.1
        and is_order_double_unit_generator_mod(g, p, h)
        and a.coprime(p) and a.congr_mod(b, p) {
        congr_mod_preserves_coprime(a, b, p)
        b.coprime(p)
        unit_quadratic_residue_iff_discrete_log_even(p, h, g, b)
    }
}

/// A product of units modulo an odd prime is a square exactly when the reduced
/// sum of their discrete logarithms is even.
theorem unit_product_quadratic_residue_iff_discrete_log_sum_even(
    p: Nat, h: Nat, g: Nat, a: Nat, b: Nat
) {
    p.is_prime and p = Nat.2 * h + Nat.1
        and is_order_double_unit_generator_mod(g, p, h)
        and a.coprime(p) and b.coprime(p)
        implies (is_unit_quadratic_residue_mod(a * b, p) = Nat.2.divides(
            (discrete_log_mod_with_order(g, p, Nat.2 * h, a) +
                discrete_log_mod_with_order(g, p, Nat.2 * h, b)).mod(Nat.2 * h)))
} by {
    if p.is_prime and p = Nat.2 * h + Nat.1
        and is_order_double_unit_generator_mod(g, p, h)
        and a.coprime(p) and b.coprime(p) {
        coprime_mul(p, a, b)
        (a * b).coprime(p)
        unit_quadratic_residue_iff_discrete_log_even(p, h, g, a * b)
        is_order_double_unit_generator_mod(g, p, h) =
            (p != Nat.0 and g.coprime(p) and h != Nat.0 and
                is_multiplicative_order_mod(g, p, Nat.2 * h) and
                powers_cover_units_mod(g, p))
        powers_cover_units_mod_apply(g, p, a)
        powers_cover_units_mod_apply(g, p, b)
        is_power_of_mod(a, g, p)
        is_power_of_mod(b, g, p)
        discrete_log_mod_with_order_mul(g, p, Nat.2 * h, a, b)
        discrete_log_mod_with_order(g, p, Nat.2 * h, a * b) =
            (discrete_log_mod_with_order(g, p, Nat.2 * h, a) +
                discrete_log_mod_with_order(g, p, Nat.2 * h, b)).mod(Nat.2 * h)
        (is_unit_quadratic_residue_mod(a * b, p) = Nat.2.divides(
            (discrete_log_mod_with_order(g, p, Nat.2 * h, a) +
                discrete_log_mod_with_order(g, p, Nat.2 * h, b)).mod(Nat.2 * h))) = true
    }
}
