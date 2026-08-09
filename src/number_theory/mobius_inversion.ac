from nat import Nat, mul_to_zero, mul_cancel_left, divides_trans,
    divides_self, divides_symm, divides_lte, lte_antisymm, lt_not_ref,
    zero_or_suc, alt_suc_ne_zero, lt_and_lte, lte_ref, lte_trans,
    add_suc_left, divides_mul, lte_add_left, lt_suc, has_prime_divisor,
    add_zero_right, add_suc_right, lt_imp_lte_suc, add_comm, mul_comm, lte_mul_both,
    gcd_of_prime, pos_of_ne_zero
from int import Int, neg_neg, add_neg, mul_one_right,
    mul_zero_right, neg_zero, one_neq_zero
from list import List, map, sum, product, is_permutation,
    permutation_preserves_length, unique_same_contains_imp_permutation,
    unique_same_contains_map_sum_eq
from list import map_map, map_length, map_contains, map_contains_of_contains,
    product_append, product_remove_one, injective_map_is_unique, add_contains_left,
    add_contains_right, add_contains_or, add_length, filter_contains_and,
    filter_contained_by_and, filter_equivalent_to_and,
    filter_only_removes_elems, unique_implies_tail_unique, unique_implies_no_duplicate,
    not_unique_implies_duplicate, unique_list_sum, unique_is_smallest_containing_list,
    sum_map_of_pointwise, map_sum_add, sum_add, sum_scalar_mul,
    list_contains_implies_count_geq_one, not_contains_add,
    cons_unique_of_tail_unique_not_contains, remove_one_unique,
    remove_one_unique_not_contains_self, filter_preserves_unique,
    list_not_contains_impl_count_zero
from data.basic.functions import is_injective_fn, compose, function_extensionality
from algebra.ring.ring import Ring, alternating_sign, alternating_sign_zero,
    alternating_sign_suc, alternating_sign_suc_mul, alternating_sign_eq_neg_one_pow,
    alternating_sign_parity, mul_neg_left, mul_zero_left, mul_neg_one_left
from number_theory.factorisation import prime_factorisation, prime_factorisation_product,
    prime_factorisation_all_prime, prime_factorisation_zero, all_prime, all_prime_nil,
    all_prime_cons_intro, all_prime_cons_elim, all_prime_append, all_prime_product_nonzero,
    all_prime_product_eq_one_imp_nil, prime_divides_product_imp_contains,
    prime_factorisation_unique, count_prime_factor, count_prime_factor_invariant,
    count_prime_factor_mul, count_prime_factor_one, count_prime_factor_self,
    count_prime_factor_non_prime, count_prime_factor_ext, divides_imp_count_prime_factor_le,
    coprime_imp_no_shared_prime_factor, all_prime_only_primes, all_prime_remove_one
from number_theory.coprime import coprime_comm, coprime_one_left, coprime_one_right,
    coprime_zero_left_imp_one, coprime_zero_right_imp_one, coprime_divides_of_divides_mul,
    coprime_of_divisors, coprime_pow_right, coprime_mul_imp_left, coprime_mul_imp_right
from number_theory.fermat import prime_divides_mul
from number_theory.divisor_sum import divisor_list, divisor_list_contains_implies,
    divisor_list_contains_of, divisor_list_is_unique, divisors_up_to_member,
    divisor_list_one, divisor_list_zero
from number_theory.dirichlet import divisor_quotient, divisor_quotient_cofactor,
    divisor_quotient_divides, divisor_quotient_positive, divisor_of_positive_is_positive
from number_theory.mobius_sums import sum_filter_split2, sum_filter_zero_removed,
    int_sum_map_neg, remove_one_length_suc, sum_split_by_p, not_divides_pred,
    divides_pred
numerals Nat
numerals Int

/// The Möbius function `mu(n)`: `1` if `n = 1`, `(-1)^k` if `n` is the product
/// of `k` distinct primes, and `0` otherwise.
define nat_mobius(n: Nat) -> Int {
    if n = Nat.0 {
        Int.0
    } else {
        if prime_factorisation(n).is_unique {
            alternating_sign[Int](prime_factorisation(n).length)
        } else {
            Int.0
        }
    }
}

/// `mu(0) = 0`.
theorem nat_mobius_zero {
    nat_mobius(Nat.0) = Int.0
} by {
    nat_mobius(Nat.0) = Int.0
}

/// The count of an element in a list never exceeds the length of the list.
theorem count_le_length[T](list: List[T], item: T) {
    list.count(item) <= list.length
} by {
    define p(l: List[T], x: T) -> Bool { l.count(x) <= l.length }
    p(List.nil[T], item) = (List.nil[T].count(item) <= List.nil[T].length)
    List.nil[T].count(item) = Nat.0
    List.nil[T].length = Nat.0
    lte_ref(Nat.0)
    Nat.0 <= Nat.0
    p(List.nil[T], item)
    forall(head: T, tail: List[T]) {
        if p(tail, item) {
            p(tail, item) = (tail.count(item) <= tail.length)
            if head = item {
                List.cons(head, tail).count(item) = Nat.1 + tail.count(item)
                List.cons(head, tail).length = tail.length.suc
                Nat.1 + tail.count(item) <= Nat.1 + tail.length
                Nat.1 + tail.length = tail.length.suc
                Nat.1 + tail.count(item) <= tail.length.suc
                List.cons(head, tail).count(item) <= tail.length.suc
                List.cons(head, tail).count(item) <= List.cons(head, tail).length
            } else {
                List.cons(head, tail).count(item) = tail.count(item)
                tail.count(item) <= tail.length
                List.cons(head, tail).count(item) <= tail.length
                tail.length <= tail.length.suc
                lte_trans(List.cons(head, tail).count(item), tail.length, tail.length.suc)
                List.cons(head, tail).count(item) <= tail.length.suc
                List.cons(head, tail).length = tail.length.suc
                List.cons(head, tail).count(item) <= List.cons(head, tail).length
            }
            p(List.cons(head, tail), item) =
                (List.cons(head, tail).count(item) <= List.cons(head, tail).length)
            p(List.cons(head, tail), item)
        }
    }
    forall(head: T, tail: List[T]) {
        p(tail, item) implies p(List.cons(head, tail), item)
    }
    p(List.nil[T], item) and forall(head: T, tail: List[T]) {
        p(tail, item) implies p(List.cons(head, tail), item)
    }
    List.induction(function(l: List[T]) { p(l, item) })
    forall(l: List[T]) { p(l, item) }
    p(list, item)
}

/// A list of length zero is the empty list.
theorem list_length_zero_imp_nil[T](list: List[T]) {
    list.length = Nat.0 implies list = List.nil[T]
} by {
    if list.length = Nat.0 {
        match list {
            List.nil {
            }
            List.cons(head, tail) {
                List.cons(head, tail).length = tail.length.suc
                tail.length.suc = Nat.0
                false
            }
        }
    }
}

/// A list of length one is unique.
theorem unique_of_length_one[T](list: List[T]) {
    list.length = Nat.1 implies list.is_unique
} by {
    if list.length = Nat.1 {
        if not list.is_unique {
            not_unique_implies_duplicate(list)
            let x: T satisfy { list.count(x) > Nat.1 }
            count_le_length(list, x)
            list.count(x) <= list.length
            list.count(x) <= Nat.1
            list.count(x) > Nat.1
            false
        }
        list.is_unique
    }
}

/// A permutation of a singleton list is the singleton list.
theorem singleton_permutation_eq[T](x: T, list: List[T]) {
    is_permutation(List.singleton(x), list) implies list = List.singleton(x)
} by {
    if is_permutation(List.singleton(x), list) {
        is_permutation(List.singleton(x), list) =
            forall(y: T) { List.singleton(x).count(y) = list.count(y) }
        permutation_preserves_length(List.singleton(x), list)
        List.singleton(x).length = list.length
        List.singleton(x) = List.cons(x, List.nil[T])
        List.cons(x, List.nil[T]).length = List.nil[T].length.suc
        List.nil[T].length = Nat.0
        List.singleton(x).length = Nat.0.suc
        Nat.0.suc = Nat.1
        List.singleton(x).length = Nat.1
        list.length = Nat.1
        match list {
            List.nil {
                List.nil[T].length = Nat.0
                false
            }
            List.cons(head, tail) {
                List.cons(head, tail).length = tail.length.suc
                list.length = tail.length.suc
                tail.length.suc = Nat.1
                tail.length = Nat.0
                list_length_zero_imp_nil(tail)
                tail = List.nil[T]
                list = List.cons(head, List.nil[T])
                List.cons(head, List.nil[T]) = List.singleton(head)
                list = List.singleton(head)
                List.singleton(x).count(head) = list.count(head)
                List.singleton(x).count(head) = List.singleton(head).count(head)
                List.singleton(head) = List.cons(head, List.nil[T])
                List.cons(head, List.nil[T]).count(head) =
                    Nat.1 + List.nil[T].count(head)
                List.nil[T].count(head) = Nat.0
                List.cons(head, List.nil[T]).count(head) = Nat.1 + Nat.0
                Nat.1 + Nat.0 = Nat.1
                List.singleton(head).count(head) = Nat.1
                List.singleton(x).count(head) = Nat.1
                if x != head {
                    List.singleton(x).count(head) = List.cons(x, List.nil[T]).count(head)
                    List.cons(x, List.nil[T]).count(head) = List.nil[T].count(head)
                    List.nil[T].count(head) = Nat.0
                    List.singleton(x).count(head) = Nat.0
                    false
                }
                x = head
                list = List.singleton(x)
            }
        }
    }
}

/// The factorisation of one is the empty list.
theorem prime_factorisation_one {
    prime_factorisation(Nat.1) = List.nil[Nat]
} by {
    Nat.1 <= Nat.1
    prime_factorisation_product(Nat.1)
    product[Nat](prime_factorisation(Nat.1)) = Nat.1
    prime_factorisation_all_prime(Nat.1)
    all_prime(prime_factorisation(Nat.1))
    all_prime_product_eq_one_imp_nil(prime_factorisation(Nat.1))
    prime_factorisation(Nat.1) = List.nil[Nat]
}

/// The factorisation of a prime is the singleton list.
theorem prime_factorisation_prime(p: Nat) {
    p.is_prime implies prime_factorisation(p) = List.singleton(p)
} by {
    if p.is_prime {
        Nat.1 < p
        Nat.1 <= p
        all_prime_nil
        all_prime_cons_intro(p, List.nil[Nat])
        all_prime(List.cons(p, List.nil[Nat]))
        product[Nat](List.cons(p, List.nil[Nat])) = p * product[Nat](List.nil[Nat])
        product[Nat](List.nil[Nat]) = Nat.1
        p * Nat.1 = p
        product[Nat](List.cons(p, List.nil[Nat])) = p
        prime_factorisation_product(p)
        product[Nat](prime_factorisation(p)) = p
        prime_factorisation_all_prime(p)
        all_prime(prime_factorisation(p))
        prime_factorisation_unique(List.cons(p, List.nil[Nat]), prime_factorisation(p))
        is_permutation(List.cons(p, List.nil[Nat]), prime_factorisation(p))
        List.cons(p, List.nil[Nat]) = List.singleton(p)
        is_permutation(List.singleton(p), prime_factorisation(p))
        singleton_permutation_eq(p, prime_factorisation(p))
        prime_factorisation(p) = List.singleton(p)
    }
}

/// `mu(1) = 1`.
theorem nat_mobius_one {
    nat_mobius(Nat.1) = Int.1
} by {
    Nat.1 != Nat.0
    prime_factorisation_one
    prime_factorisation(Nat.1) = List.nil[Nat]
    List.nil[Nat].is_unique
    alternating_sign_zero[Int]
    alternating_sign[Int](Nat.0) = Int.1
    List.nil[Nat].length = Nat.0
    nat_mobius(Nat.1) = alternating_sign[Int](prime_factorisation(Nat.1).length)
    alternating_sign[Int](prime_factorisation(Nat.1).length) = Int.1
}

/// `mu(p) = -1` for primes `p`.
theorem nat_mobius_prime(p: Nat) {
    p.is_prime implies nat_mobius(p) = -Int.1
} by {
    if p.is_prime {
        Nat.1 < p
        Nat.0 < Nat.1
        lt_and_lte(Nat.0, Nat.1, p)
        Nat.0 < p
        p != Nat.0
        prime_factorisation_prime(p)
        prime_factorisation(p) = List.singleton(p)
        List.singleton(p).is_unique
        List.singleton(p).length = Nat.1
        alternating_sign_suc[Int](Nat.0)
        alternating_sign[Int](Nat.0.suc) = -alternating_sign[Int](Nat.0)
        alternating_sign_zero[Int]
        alternating_sign[Int](Nat.0) = Int.1
        Nat.0.suc = Nat.1
        alternating_sign[Int](Nat.1) = -Int.1
        nat_mobius(p) = alternating_sign[Int](prime_factorisation(p).length)
        alternating_sign[Int](prime_factorisation(p).length) = alternating_sign[Int](Nat.1)
        nat_mobius(p) = -Int.1
    }
}

/// A member of a list of natural numbers divides the product of the list.
theorem list_member_divides_product(l: List[Nat], x: Nat) {
    l.contains(x) implies x.divides(product[Nat](l))
} by {
    define p(ls: List[Nat]) -> Bool {
        ls.contains(x) implies x.divides(product[Nat](ls))
    }
    p(List.nil[Nat]) = (List.nil[Nat].contains(x) implies x.divides(product[Nat](List.nil[Nat])))
    not List.nil[Nat].contains(x)
    p(List.nil[Nat])
    forall(head: Nat, tail: List[Nat]) {
        if p(tail) {
            p(tail) = (tail.contains(x) implies x.divides(product[Nat](tail)))
            if List.cons(head, tail).contains(x) {
                if x = head {
                    product[Nat](List.cons(head, tail)) = head * product[Nat](tail)
                    x * product[Nat](tail) = head * product[Nat](tail)
                    x.divides(product[Nat](List.cons(head, tail)))
                } else {
                    x != head
                    tail.contains(x)
                    x.divides(product[Nat](tail))
                    divides_mul(head, product[Nat](tail), x)
                    x.divides(product[Nat](tail) * head)
                    product[Nat](tail) * head = head * product[Nat](tail)
                    x.divides(head * product[Nat](tail))
                    product[Nat](List.cons(head, tail)) = head * product[Nat](tail)
                    x.divides(product[Nat](List.cons(head, tail)))
                }
            }
            p(List.cons(head, tail)) =
                (List.cons(head, tail).contains(x) implies x.divides(product[Nat](List.cons(head, tail))))
            p(List.cons(head, tail))
        }
    }
    forall(head: Nat, tail: List[Nat]) {
        p(tail) implies p(List.cons(head, tail))
    }
    p(List.nil[Nat]) and forall(head: Nat, tail: List[Nat]) {
        p(tail) implies p(List.cons(head, tail))
    }
    List.induction(p)
    forall(ls: List[Nat]) { p(ls) }
    p(l)
}

/// The append-uniqueness predicate: no element of the left list lies in `l2`.
define append_unique_no_shared_pred[T](l2: List[T]) -> (List[T] -> Bool) {
    function(ls: List[T]) {
        ls.is_unique and l2.is_unique and
            (forall(x: T) { ls.contains(x) implies not l2.contains(x) })
            implies (ls + l2).is_unique
    }
}

/// Base case: the empty list concatenated with a unique list is unique.
theorem append_unique_of_no_shared_base[T](l2: List[T]) {
    (List.nil[T].is_unique and l2.is_unique and
        (forall(x: T) { List.nil[T].contains(x) implies not l2.contains(x) }))
        implies (List.nil[T] + l2).is_unique
} by {
    if List.nil[T].is_unique and l2.is_unique and
            (forall(x: T) { List.nil[T].contains(x) implies not l2.contains(x) }) {
        List.nil[T] + l2 = l2
        (List.nil[T] + l2).is_unique
    }
}

/// Step case: a fresh head keeps the concatenation unique.
theorem append_unique_of_no_shared_step[T](head: T, tail: List[T], l2: List[T]) {
    (tail.is_unique and l2.is_unique and
        (forall(x: T) { tail.contains(x) implies not l2.contains(x) })
        implies (tail + l2).is_unique) implies
        (List.cons(head, tail).is_unique and l2.is_unique and
            (forall(x: T) {
                List.cons(head, tail).contains(x) implies not l2.contains(x)
            })
            implies (List.cons(head, tail) + l2).is_unique)
} by {
    if tail.is_unique and l2.is_unique and
            (forall(x: T) { tail.contains(x) implies not l2.contains(x) })
            implies (tail + l2).is_unique {
        if List.cons(head, tail).is_unique and l2.is_unique and
                (forall(x: T) {
                    List.cons(head, tail).contains(x) implies not l2.contains(x)
                }) {
            unique_implies_tail_unique(head, tail)
            tail.is_unique
            forall(x: T) {
                if tail.contains(x) {
                    List.cons(head, tail).contains(x)
                    not l2.contains(x)
                }
            }
            (tail + l2).is_unique
            List.cons(head, tail).contains(head)
            not l2.contains(head)
            if tail.contains(head) {
                List.cons(head, tail).count(head) =
                    Nat.1 + tail.count(head)
                list_contains_implies_count_geq_one(tail, head)
                tail.count(head) >= Nat.1
                unique_implies_no_duplicate(List.cons(head, tail), head)
                List.cons(head, tail).count(head) <= Nat.1
                Nat.1 + tail.count(head) <= Nat.1
                lte_add_left(Nat.1, Nat.1, tail.count(head))
                Nat.1 + Nat.1 <= Nat.1 + tail.count(head)
                Nat.1 + Nat.1 = Nat.2
                Nat.2 <= Nat.1 + tail.count(head)
                lte_trans(Nat.2, Nat.1 + tail.count(head), Nat.1)
                Nat.2 <= Nat.1
                lt_suc(Nat.1)
                Nat.1 < Nat.2
                lt_and_lte(Nat.1, Nat.2, Nat.1)
                Nat.1 < Nat.1
                lt_not_ref(Nat.1)
                false
            }
            not tail.contains(head)
            not_contains_add(tail, l2, head)
            not (tail + l2).contains(head)
            cons_unique_of_tail_unique_not_contains(head, tail + l2)
            List.cons(head, tail + l2).is_unique
            List.cons(head, tail) + l2 = List.cons(head, tail + l2)
            (List.cons(head, tail) + l2).is_unique
        }
    }
}

/// The concatenation of two unique lists with no shared elements is unique.
theorem append_unique_of_no_shared[T](l1: List[T], l2: List[T]) {
    l1.is_unique and l2.is_unique and
        (forall(x: T) { l1.contains(x) implies not l2.contains(x) })
        implies (l1 + l2).is_unique
} by {
    append_unique_of_no_shared_base(l2)
    forall(head: T, tail: List[T]) {
        if append_unique_no_shared_pred(l2)(tail) {
            append_unique_of_no_shared_step(head, tail, l2)
        }
    }
    append_unique_no_shared_pred(l2)(l1)
}

/// A permutation of a unique list is unique.
theorem permutation_of_unique_is_unique[T](a: List[T], b: List[T]) {
    is_permutation(a, b) and a.is_unique implies b.is_unique
} by {
    if is_permutation(a, b) and a.is_unique {
        is_permutation(a, b) = forall(x: T) { a.count(x) = b.count(x) }
        if not b.is_unique {
            not_unique_implies_duplicate(b)
            let x: T satisfy { b.count(x) > Nat.1 }
            b.count(x) = a.count(x)
            unique_implies_no_duplicate(a, x)
            a.count(x) <= Nat.1
            b.count(x) <= Nat.1
            false
        }
        b.is_unique
    }
}

/// The alternating sign of a sum is the product of the signs.
theorem alternating_sign_add[R: Ring](m: Nat, n: Nat) {
    alternating_sign[R](m + n) = alternating_sign[R](m) * alternating_sign[R](n)
} by {
    define p(k: Nat) -> Bool {
        alternating_sign[R](k + n) = alternating_sign[R](k) * alternating_sign[R](n)
    }
    alternating_sign_zero[R]
    alternating_sign[R](Nat.0) = R.1
    alternating_sign[R](Nat.0 + n) = alternating_sign[R](Nat.0) * alternating_sign[R](n)
    p(Nat.0)
    forall(k: Nat) {
        if p(k) {
            p(k) = (alternating_sign[R](k + n) = alternating_sign[R](k) * alternating_sign[R](n))
            alternating_sign_suc[R](k + n)
            alternating_sign[R]((k + n).suc) = -alternating_sign[R](k + n)
            add_suc_left(k, n)
            k.suc + n = (k + n).suc
            alternating_sign[R](k.suc + n) = -alternating_sign[R](k + n)
            alternating_sign_suc[R](k)
            alternating_sign[R](k.suc) = -alternating_sign[R](k)
            mul_neg_left[R](alternating_sign[R](k), alternating_sign[R](n))
            -alternating_sign[R](k) * alternating_sign[R](n) =
                -(alternating_sign[R](k) * alternating_sign[R](n))
            alternating_sign[R](k.suc) * alternating_sign[R](n) =
                -(alternating_sign[R](k) * alternating_sign[R](n))
            alternating_sign[R](k.suc + n) =
                alternating_sign[R](k.suc) * alternating_sign[R](n)
            p(k.suc) = (alternating_sign[R](k.suc + n) =
                alternating_sign[R](k.suc) * alternating_sign[R](n))
            p(k.suc)
        }
    }
    p(Nat.0) and forall(k: Nat) {
        p(k) implies p(k.suc)
    }
    Nat.induction(p)
    p(m)
}

/// The alternating sign never vanishes on the integers.
theorem int_alternating_sign_nonzero(k: Nat) {
    alternating_sign[Int](k) != Int.0
} by {
    define p(n: Nat) -> Bool { alternating_sign[Int](n) != Int.0 }
    alternating_sign_zero[Int]
    alternating_sign[Int](Nat.0) = Int.1
    one_neq_zero
    Int.1 != Int.0
    p(Nat.0)
    forall(n: Nat) {
        if p(n) {
            alternating_sign_suc[Int](n)
            alternating_sign[Int](n.suc) = -alternating_sign[Int](n)
            if alternating_sign[Int](n.suc) = Int.0 {
                -alternating_sign[Int](n) = Int.0
                neg_neg(alternating_sign[Int](n))
                -(-alternating_sign[Int](n)) = alternating_sign[Int](n)
                neg_zero
                -Int.0 = Int.0
                -(-alternating_sign[Int](n)) = -Int.0
                alternating_sign[Int](n) = Int.0
                false
            }
            alternating_sign[Int](n.suc) != Int.0
            p(n.suc)
        }
    }
    p(Nat.0) and forall(n: Nat) {
        p(n) implies p(n.suc)
    }
    Nat.induction(p)
    p(k)
}

/// `mu(n) != 0` iff the factorisation of `n` is a unique list, for positive `n`.
theorem nat_mobius_nonzero_iff_factorisation_unique(n: Nat) {
    Nat.0 < n implies (nat_mobius(n) != Int.0) = prime_factorisation(n).is_unique
} by {
    if Nat.0 < n {
        n != Nat.0
        if prime_factorisation(n).is_unique {
            nat_mobius(n) = alternating_sign[Int](prime_factorisation(n).length)
            int_alternating_sign_nonzero(prime_factorisation(n).length)
            alternating_sign[Int](prime_factorisation(n).length) != Int.0
            nat_mobius(n) != Int.0
        } else {
            not prime_factorisation(n).is_unique
            nat_mobius(n) = Int.0
            if nat_mobius(n) = Int.0 {
            }
            not (nat_mobius(n) != Int.0)
        }
        (nat_mobius(n) != Int.0) = prime_factorisation(n).is_unique
    }
}

/// A natural number is at most its sum with another.
theorem nat_le_add_right(x: Nat, y: Nat) {
    x <= x + y
} by {
    define p(k: Nat) -> Bool { x <= x + k }
    add_zero_right(x)
    x + Nat.0 = x
    lte_ref(x)
    x <= x
    p(Nat.0)
    forall(k: Nat) {
        if p(k) {
            p(k) = (x <= x + k)
            lt_suc(x + k)
            x + k < (x + k).suc
            add_suc_right(x, k)
            x + k.suc = (x + k).suc
            x + k < x + k.suc
            lt_imp_lte_suc(x + k, x + k.suc)
            (x + k).suc <= x + k.suc
            x + k <= x + k.suc
            lte_trans(x, x + k, x + k.suc)
            x <= x + k.suc
            p(k.suc)
        }
    }
    p(Nat.0) and forall(k: Nat) {
        p(k) implies p(k.suc)
    }
    Nat.induction(p)
    p(y)
}

/// If the left factor is not squarefree, the product has zero Möbius value.
theorem nat_mobius_mul_left_nonunique(a: Nat, b: Nat) {
    a != Nat.0 and b != Nat.0 and not prime_factorisation(a).is_unique
        implies nat_mobius(a * b) = nat_mobius(a) * nat_mobius(b)
} by {
    if a != Nat.0 and b != Nat.0 and not prime_factorisation(a).is_unique {
        if a * b = Nat.0 {
            mul_to_zero(a, b)
            a = Nat.0 or b = Nat.0
            false
        }
        a * b != Nat.0
        not_unique_implies_duplicate(prime_factorisation(a))
        let q: Nat satisfy { prime_factorisation(a).count(q) > Nat.1 }
        prime_factorisation(a).count(q) = count_prime_factor(q, a)
        count_prime_factor(q, a) > Nat.1
        count_prime_factor_mul(q, a, b)
        count_prime_factor(q, a * b) = count_prime_factor(q, a) + count_prime_factor(q, b)
        nat_le_add_right(count_prime_factor(q, a), count_prime_factor(q, b))
        count_prime_factor(q, a) <= count_prime_factor(q, a) + count_prime_factor(q, b)
        count_prime_factor(q, a) <= count_prime_factor(q, a * b)
        count_prime_factor(q, a) > Nat.1
        lt_and_lte(Nat.1, count_prime_factor(q, a), count_prime_factor(q, a * b))
        Nat.1 < count_prime_factor(q, a * b)
        count_prime_factor(q, a * b) > Nat.1 = (Nat.1 < count_prime_factor(q, a * b))
        count_prime_factor(q, a * b) > Nat.1
        prime_factorisation(a * b).count(q) = count_prime_factor(q, a * b)
        prime_factorisation(a * b).count(q) > Nat.1
        if prime_factorisation(a * b).is_unique {
            unique_implies_no_duplicate(prime_factorisation(a * b), q)
            prime_factorisation(a * b).count(q) <= Nat.1
            false
        }
        not prime_factorisation(a * b).is_unique
        nat_mobius(a) = Int.0
        nat_mobius(a * b) = Int.0
        nat_mobius(a) * nat_mobius(b) = Int.0 * nat_mobius(b)
        mul_zero_left[Int](nat_mobius(b))
        Int.0 * nat_mobius(b) = Int.0
        nat_mobius(a * b) = nat_mobius(a) * nat_mobius(b)
    }
}

/// If the right factor is not squarefree, the product has zero Möbius value.
theorem nat_mobius_mul_right_nonunique(a: Nat, b: Nat) {
    a != Nat.0 and b != Nat.0 and not prime_factorisation(b).is_unique
        implies nat_mobius(a * b) = nat_mobius(a) * nat_mobius(b)
} by {
    if a != Nat.0 and b != Nat.0 and not prime_factorisation(b).is_unique {
        if a * b = Nat.0 {
            mul_to_zero(a, b)
            a = Nat.0 or b = Nat.0
            false
        }
        a * b != Nat.0
        not_unique_implies_duplicate(prime_factorisation(b))
        let q: Nat satisfy { prime_factorisation(b).count(q) > Nat.1 }
        prime_factorisation(b).count(q) = count_prime_factor(q, b)
        count_prime_factor(q, b) > Nat.1
        count_prime_factor_mul(q, a, b)
        count_prime_factor(q, a * b) = count_prime_factor(q, a) + count_prime_factor(q, b)
        nat_le_add_right(count_prime_factor(q, b), count_prime_factor(q, a))
        count_prime_factor(q, b) <= count_prime_factor(q, b) + count_prime_factor(q, a)
        add_comm(count_prime_factor(q, a), count_prime_factor(q, b))
        count_prime_factor(q, a) + count_prime_factor(q, b) =
            count_prime_factor(q, b) + count_prime_factor(q, a)
        count_prime_factor(q, b) <= count_prime_factor(q, a * b)
        count_prime_factor(q, b) > Nat.1
        lt_and_lte(Nat.1, count_prime_factor(q, b), count_prime_factor(q, a * b))
        Nat.1 < count_prime_factor(q, a * b)
        count_prime_factor(q, a * b) > Nat.1 = (Nat.1 < count_prime_factor(q, a * b))
        count_prime_factor(q, a * b) > Nat.1
        prime_factorisation(a * b).count(q) = count_prime_factor(q, a * b)
        prime_factorisation(a * b).count(q) > Nat.1
        if prime_factorisation(a * b).is_unique {
            unique_implies_no_duplicate(prime_factorisation(a * b), q)
            prime_factorisation(a * b).count(q) <= Nat.1
            false
        }
        not prime_factorisation(a * b).is_unique
        nat_mobius(b) = Int.0
        nat_mobius(a * b) = Int.0
        nat_mobius(a) * nat_mobius(b) = nat_mobius(a) * Int.0
        mul_zero_right(nat_mobius(a))
        nat_mobius(a) * Int.0 = Int.0
        nat_mobius(a * b) = nat_mobius(a) * nat_mobius(b)
    }
}

/// The Möbius value of a product of squarefree coprime factors.
theorem nat_mobius_mul_unique(a: Nat, b: Nat) {
    a != Nat.0 and b != Nat.0 and prime_factorisation(a).is_unique and
        prime_factorisation(b).is_unique and a.coprime(b)
        implies nat_mobius(a * b) = nat_mobius(a) * nat_mobius(b)
} by {
    if a != Nat.0 and b != Nat.0 and prime_factorisation(a).is_unique and
            prime_factorisation(b).is_unique and a.coprime(b) {
        if a * b = Nat.0 {
            mul_to_zero(a, b)
            a = Nat.0 or b = Nat.0
            false
        }
        a * b != Nat.0
        Nat.1 <= a
        Nat.1 <= b
        Nat.1 <= a * b
        let fa: List[Nat] = prime_factorisation(a)
        let fb: List[Nat] = prime_factorisation(b)
        let fab: List[Nat] = prime_factorisation(a * b)
        prime_factorisation_product(a)
        prime_factorisation_product(b)
        prime_factorisation_product(a * b)
        prime_factorisation_all_prime(a)
        prime_factorisation_all_prime(b)
        prime_factorisation_all_prime(a * b)
        all_prime(fa)
        all_prime(fb)
        all_prime(fab)
        product[Nat](fa) = a
        product[Nat](fb) = b
        product[Nat](fab) = a * b
        fa.is_unique
        fb.is_unique
        nat_mobius(a) = alternating_sign[Int](fa.length)
        nat_mobius(b) = alternating_sign[Int](fb.length)
        let combined: List[Nat] = fa + fb
        all_prime_append(fa, fb)
        all_prime(combined)
        product_append[Nat](fa, fb)
        product[Nat](combined) = product[Nat](fa) * product[Nat](fb)
        product[Nat](combined) = a * b
        prime_factorisation_unique(combined, fab)
        is_permutation(combined, fab)
        permutation_preserves_length(combined, fab)
        combined.length = fab.length
        add_length(fa, fb)
        combined.length = fa.length + fb.length
        fab.length = fa.length + fb.length
        forall(x: Nat) {
            if fa.contains(x) {
                list_member_divides_product(fa, x)
                x.divides(a)
                all_prime_only_primes(fa, x)
                x.is_prime
                if fb.contains(x) {
                    list_member_divides_product(fb, x)
                    x.divides(b)
                    coprime_imp_no_shared_prime_factor(a, b, x)
                    a.coprime(b) and x.is_prime implies not (x.divides(a) and x.divides(b))
                    a.coprime(b)
                    a.coprime(b) and x.is_prime
                    not (x.divides(a) and x.divides(b))
                    x.divides(a) and x.divides(b)
                    false
                }
                not fb.contains(x)
            }
        }
        append_unique_of_no_shared(fa, fb)
        combined.is_unique
        permutation_of_unique_is_unique(combined, fab)
        fab.is_unique
        nat_mobius(a * b) = alternating_sign[Int](fab.length)
        alternating_sign_add[Int](fa.length, fb.length)
        alternating_sign[Int](fa.length + fb.length) =
            alternating_sign[Int](fa.length) * alternating_sign[Int](fb.length)
        alternating_sign[Int](fab.length) =
            alternating_sign[Int](fa.length) * alternating_sign[Int](fb.length)
        nat_mobius(a * b) = nat_mobius(a) * nat_mobius(b)
    }
}

/// The Möbius function is multiplicative on coprime arguments.
theorem nat_mobius_multiplicative_apply(a: Nat, b: Nat) {
    a.coprime(b) implies nat_mobius(a * b) = nat_mobius(a) * nat_mobius(b)
} by {
    if a.coprime(b) {
        if a = Nat.0 {
            coprime_zero_left_imp_one(b)
            b = Nat.1
            a * b = Nat.0
            Nat.0 * b = Nat.0
            nat_mobius(a * b) = nat_mobius(Nat.0)
            nat_mobius_zero
            nat_mobius(Nat.0) = Int.0
            nat_mobius(a) = Int.0
            nat_mobius_one
            nat_mobius(b) = Int.1
            nat_mobius(a) * nat_mobius(b) = Int.0 * Int.1
            mul_zero_left[Int](Int.1)
            Int.0 * Int.1 = Int.0
            nat_mobius(a * b) = nat_mobius(a) * nat_mobius(b)
        } else {
            if b = Nat.0 {
                coprime_zero_right_imp_one(a)
                a = Nat.1
                a * b = Nat.0
                nat_mobius(a * b) = nat_mobius(Nat.0)
                nat_mobius_zero
                nat_mobius(Nat.0) = Int.0
                nat_mobius(b) = Int.0
                nat_mobius_one
                nat_mobius(a) = Int.1
                nat_mobius(a) * nat_mobius(b) = Int.1 * Int.0
                mul_zero_right(Int.1)
                Int.1 * Int.0 = Int.0
                nat_mobius(a * b) = nat_mobius(a) * nat_mobius(b)
            } else {
                a != Nat.0
                b != Nat.0
                if prime_factorisation(a).is_unique {
                    if prime_factorisation(b).is_unique {
                        nat_mobius_mul_unique(a, b)
                        nat_mobius(a * b) = nat_mobius(a) * nat_mobius(b)
                    } else {
                        nat_mobius_mul_right_nonunique(a, b)
                        nat_mobius(a * b) = nat_mobius(a) * nat_mobius(b)
                    }
                } else {
                    nat_mobius_mul_left_nonunique(a, b)
                    nat_mobius(a * b) = nat_mobius(a) * nat_mobius(b)
                }
            }
        }
    }
}

/// The Möbius support predicate: `mu(d) != 0`.
define nat_mobius_support(d: Nat) -> Bool {
    nat_mobius(d) != Int.0
}

/// Multiplication by a fixed natural.
define mul_by_nat(p: Nat) -> (Nat -> Nat) {
    function(x: Nat) { p * x }
}

/// Removing one occurrence of a prime from a squarefree natural keeps the
/// Möbius support and flips the sign.
theorem nat_mobius_remove_prime(x: Nat, p: Nat) {
    Nat.0 < x and p.is_prime and p.divides(x) and prime_factorisation(x).is_unique
        implies
            nat_mobius(divisor_quotient(x, p)) != Int.0
            and not p.divides(divisor_quotient(x, p))
            and nat_mobius(x) = -nat_mobius(divisor_quotient(x, p))
} by {
    if Nat.0 < x and p.is_prime and p.divides(x) and prime_factorisation(x).is_unique {
        let fx: List[Nat] = prime_factorisation(x)
        let rest: List[Nat] = fx.remove_one(p)
        let a: Nat = divisor_quotient(x, p)
        Nat.1 <= x
        prime_factorisation_product(x)
        product[Nat](fx) = x
        prime_factorisation_all_prime(x)
        all_prime(fx)
        fx.is_unique
        p.divides(product[Nat](fx))
        prime_divides_product_imp_contains(fx, p)
        fx.contains(p)
        all_prime_remove_one(fx, p)
        all_prime(rest)
        product_remove_one[Nat](fx, p)
        p * product[Nat](rest) = product[Nat](fx)
        p * product[Nat](rest) = x
        divisor_quotient_cofactor(x, p)
        p * divisor_quotient(x, p) = x
        p * a = x
        p * product[Nat](rest) = p * a
        p != Nat.0
        mul_cancel_left(p, product[Nat](rest), a)
        product[Nat](rest) = a
        if a = Nat.0 {
            p * Nat.0 = Nat.0
            p * a = Nat.0
            x = Nat.0
            false
        }
        a != Nat.0
        Nat.1 <= a
        let fa: List[Nat] = prime_factorisation(a)
        prime_factorisation_product(a)
        product[Nat](fa) = a
        prime_factorisation_all_prime(a)
        all_prime(fa)
        prime_factorisation_unique(rest, fa)
        is_permutation(rest, fa)
        remove_one_unique(fx, p)
        rest.is_unique
        permutation_of_unique_is_unique(rest, fa)
        fa.is_unique
        if p.divides(a) {
            p.divides(product[Nat](fa))
            prime_divides_product_imp_contains(fa, p)
            fa.contains(p)
            is_permutation(rest, fa) = forall(y: Nat) { rest.count(y) = fa.count(y) }
            rest.count(p) = fa.count(p)
            list_contains_implies_count_geq_one(fa, p)
            fa.count(p) >= Nat.1
            rest.count(p) >= Nat.1
            remove_one_unique_not_contains_self(fx, p)
            not rest.contains(p)
            list_not_contains_impl_count_zero(rest, p)
            rest.count(p) = Nat.0
            false
        }
        not p.divides(a)
        remove_one_length_suc(fx, p)
        rest.length.suc = fx.length
        permutation_preserves_length(rest, fa)
        rest.length = fa.length
        alternating_sign_suc[Int](rest.length)
        alternating_sign[Int](rest.length.suc) = -alternating_sign[Int](rest.length)
        alternating_sign[Int](fx.length) = -alternating_sign[Int](rest.length)
        alternating_sign[Int](fx.length) = -alternating_sign[Int](fa.length)
        nat_mobius(a) = alternating_sign[Int](fa.length)
        nat_mobius(x) = alternating_sign[Int](fx.length)
        nat_mobius(x) = -nat_mobius(a)
        int_alternating_sign_nonzero(fa.length)
        alternating_sign[Int](fa.length) != Int.0
        nat_mobius(a) != Int.0
        nat_mobius(divisor_quotient(x, p)) != Int.0
        not p.divides(divisor_quotient(x, p))
        nat_mobius(x) = -nat_mobius(divisor_quotient(x, p))
        nat_mobius(divisor_quotient(x, p)) != Int.0 and
            not p.divides(divisor_quotient(x, p)) and
            nat_mobius(x) = -nat_mobius(divisor_quotient(x, p))
    }
}

/// Forward direction of the Möbius pairing: a squarefree divisor not divisible
/// by `p` pairs with its multiple by `p`.
theorem mobius_pair_forward(n: Nat, p: Nat, d: Nat) {
    Nat.0 < n and p.is_prime and p.divides(n) and
        divisor_list(n).filter(nat_mobius_support).contains(d) and
        not p.divides(d)
        implies
            divisor_list(n).filter(nat_mobius_support).contains(p * d) and
            nat_mobius(p * d) = -nat_mobius(d)
} by {
    if Nat.0 < n and p.is_prime and p.divides(n) and
            divisor_list(n).filter(nat_mobius_support).contains(d) and
            not p.divides(d) {
        filter_contained_by_and(divisor_list(n), nat_mobius_support, d)
        divisor_list(n).contains(d) and nat_mobius_support(d)
        divisor_list(n).contains(d)
        divisor_list_contains_implies(n, d)
        Nat.0 < d and d.divides(n)
        Nat.0 < d
        d.divides(n)
        nat_mobius_support(d) = (nat_mobius(d) != Int.0)
        nat_mobius(d) != Int.0
        let c: Nat satisfy { d * c = n }
        d * c = n
        p.divides(d * c)
        prime_divides_mul(p, d, c)
        p.divides(d) or p.divides(c)
        not p.divides(d)
        p.divides(c)
        let c2: Nat satisfy { p * c2 = c }
        p * c2 = c
        d * (p * c2) = (p * d) * c2
        n = (p * d) * c2
        (p * d).divides(n)
        Nat.1 <= p
        Nat.1 <= d
        lte_mul_both(p, Nat.1, d)
        p * Nat.1 <= p * d
        p * Nat.1 = p
        Nat.1 <= p * d
        Nat.0 < p * d
        divisor_list_contains_of(n, p * d)
        divisor_list(n).contains(p * d)
        gcd_of_prime(p, d)
        p.gcd(d) = Nat.1 or p.divides(d)
        not p.divides(d)
        p.gcd(d) = Nat.1
        p.coprime(d)
        nat_mobius_multiplicative_apply(p, d)
        nat_mobius(p * d) = nat_mobius(p) * nat_mobius(d)
        nat_mobius_prime(p)
        nat_mobius(p) = -Int.1
        nat_mobius(p) * nat_mobius(d) = -Int.1 * nat_mobius(d)
        mul_neg_one_left[Int](nat_mobius(d))
        -Int.1 * nat_mobius(d) = -nat_mobius(d)
        nat_mobius(p * d) = -nat_mobius(d)
        if -nat_mobius(d) = Int.0 {
            neg_neg(nat_mobius(d))
            -(-nat_mobius(d)) = nat_mobius(d)
            neg_zero
            -Int.0 = Int.0
            -(-nat_mobius(d)) = -Int.0
            nat_mobius(d) = Int.0
            false
        }
        nat_mobius(p * d) != Int.0
        nat_mobius_support(p * d) = (nat_mobius(p * d) != Int.0)
        nat_mobius_support(p * d)
        divisor_list(n).contains(p * d) and nat_mobius_support(p * d)
        filter_contains_and(divisor_list(n), nat_mobius_support, p * d)
        divisor_list(n).filter(nat_mobius_support).contains(p * d)
        nat_mobius(p * d) = -nat_mobius(d)
        divisor_list(n).filter(nat_mobius_support).contains(p * d) and
            nat_mobius(p * d) = -nat_mobius(d)
    }
}

/// Backward direction of the Möbius pairing: a squarefree divisor divisible by
/// `p` is the multiple by `p` of the corresponding squarefree divisor.
theorem mobius_pair_backward(n: Nat, p: Nat, x: Nat) {
    Nat.0 < n and p.is_prime and p.divides(n) and
        divisor_list(n).filter(nat_mobius_support).contains(x) and
        p.divides(x)
        implies
            divisor_list(n).filter(nat_mobius_support).contains(divisor_quotient(x, p)) and
            not p.divides(divisor_quotient(x, p)) and
            x = p * divisor_quotient(x, p)
} by {
    if Nat.0 < n and p.is_prime and p.divides(n) and
            divisor_list(n).filter(nat_mobius_support).contains(x) and
            p.divides(x) {
        filter_contained_by_and(divisor_list(n), nat_mobius_support, x)
        divisor_list(n).contains(x) and nat_mobius_support(x)
        divisor_list(n).contains(x)
        divisor_list_contains_implies(n, x)
        Nat.0 < x and x.divides(n)
        Nat.0 < x
        x.divides(n)
        nat_mobius_support(x) = (nat_mobius(x) != Int.0)
        nat_mobius(x) != Int.0
        nat_mobius_nonzero_iff_factorisation_unique(x)
        prime_factorisation(x).is_unique
        nat_mobius_remove_prime(x, p)
        nat_mobius(divisor_quotient(x, p)) != Int.0
        not p.divides(divisor_quotient(x, p))
        nat_mobius(x) = -nat_mobius(divisor_quotient(x, p))
        divisor_quotient_cofactor(x, p)
        p * divisor_quotient(x, p) = x
        x = p * divisor_quotient(x, p)
        let c: Nat satisfy { x * c = n }
        x * c = n
        (p * divisor_quotient(x, p)) * c = n
        (p * divisor_quotient(x, p)) * c = divisor_quotient(x, p) * (p * c)
        divisor_quotient(x, p) * (p * c) = n
        divisor_quotient(x, p).divides(n)
        if divisor_quotient(x, p) = Nat.0 {
            p * Nat.0 = Nat.0
            p * divisor_quotient(x, p) = Nat.0
            x = Nat.0
            false
        }
        divisor_quotient(x, p) != Nat.0
        Nat.0 < divisor_quotient(x, p)
        divisor_list_contains_of(n, divisor_quotient(x, p))
        divisor_list(n).contains(divisor_quotient(x, p))
        nat_mobius_support(divisor_quotient(x, p)) =
            (nat_mobius(divisor_quotient(x, p)) != Int.0)
        nat_mobius_support(divisor_quotient(x, p))
        divisor_list(n).contains(divisor_quotient(x, p)) and
            nat_mobius_support(divisor_quotient(x, p))
        filter_contains_and(divisor_list(n), nat_mobius_support, divisor_quotient(x, p))
        divisor_list(n).filter(nat_mobius_support).contains(divisor_quotient(x, p))
        not p.divides(divisor_quotient(x, p))
        x = p * divisor_quotient(x, p)
        divisor_list(n).filter(nat_mobius_support).contains(divisor_quotient(x, p)) and
            not p.divides(divisor_quotient(x, p)) and
            x = p * divisor_quotient(x, p)
    }
}

/// The image of the `p`-free squarefree divisors under multiplication by `p`
/// permutes the `p`-divisible squarefree divisors.
theorem mobius_pair_image_permutation(n: Nat, p: Nat) {
    Nat.0 < n and p.is_prime and p.divides(n) implies
        is_permutation(
            map(divisor_list(n).filter(nat_mobius_support).filter(not_divides_pred(p)),
                mul_by_nat(p)),
            divisor_list(n).filter(nat_mobius_support).filter(divides_pred(p)))
} by {
    if Nat.0 < n and p.is_prime and p.divides(n) {
        let fs: List[Nat] = divisor_list(n).filter(nat_mobius_support)
        let la: List[Nat] = fs.filter(not_divides_pred(p))
        let lb: List[Nat] = fs.filter(divides_pred(p))
        divisor_list_is_unique(n)
        filter_preserves_unique(divisor_list(n), nat_mobius_support)
        fs.is_unique
        filter_preserves_unique(fs, not_divides_pred(p))
        la.is_unique
        filter_preserves_unique(fs, divides_pred(p))
        lb.is_unique
        is_injective_fn(mul_by_nat(p)) = forall(x: Nat, y: Nat) {
            mul_by_nat(p)(x) = mul_by_nat(p)(y) implies x = y
        }
        forall(x: Nat, y: Nat) {
            if mul_by_nat(p)(x) = mul_by_nat(p)(y) {
                mul_by_nat(p)(x) = p * x
                mul_by_nat(p)(y) = p * y
                p * x = p * y
                p != Nat.0
                mul_cancel_left(p, x, y)
                x = y
            }
        }
        is_injective_fn(mul_by_nat(p))
        injective_map_is_unique(la, mul_by_nat(p))
        map(la, mul_by_nat(p)).is_unique
        forall(y: Nat) {
            if map(la, mul_by_nat(p)).contains(y) {
                map_contains(la, mul_by_nat(p), y)
                let a: Nat satisfy { la.contains(a) and mul_by_nat(p)(a) = y }
                la.contains(a)
                mul_by_nat(p)(a) = y
                mul_by_nat(p)(a) = p * a
                y = p * a
                fs.filter(not_divides_pred(p)).contains(a)
                filter_contained_by_and(fs, not_divides_pred(p), a)
                fs.contains(a) and not_divides_pred(p)(a)
                fs.contains(a)
                not_divides_pred(p)(a) = (not p.divides(a))
                not p.divides(a)
                filter_contained_by_and(divisor_list(n), nat_mobius_support, a)
                divisor_list(n).filter(nat_mobius_support).contains(a)
                mobius_pair_forward(n, p, a)
                divisor_list(n).filter(nat_mobius_support).contains(p * a)
                divisor_list(n).filter(nat_mobius_support).contains(y)
                p.divides(y)
                divides_pred(p)(y) = p.divides(y)
                divides_pred(p)(y)
                divisor_list(n).filter(nat_mobius_support).contains(y) and divides_pred(p)(y)
                filter_contains_and(divisor_list(n), nat_mobius_support, y)
                fs.contains(y)
                filter_contains_and(fs, divides_pred(p), y)
                lb.contains(y)
            }
            if lb.contains(y) {
                fs.filter(divides_pred(p)).contains(y)
                filter_contained_by_and(fs, divides_pred(p), y)
                fs.contains(y) and divides_pred(p)(y)
                fs.contains(y)
                divides_pred(p)(y) = p.divides(y)
                p.divides(y)
                filter_contained_by_and(divisor_list(n), nat_mobius_support, y)
                divisor_list(n).filter(nat_mobius_support).contains(y)
                mobius_pair_backward(n, p, y)
                divisor_list(n).filter(nat_mobius_support).contains(divisor_quotient(y, p))
                not p.divides(divisor_quotient(y, p))
                y = p * divisor_quotient(y, p)
                let a: Nat = divisor_quotient(y, p)
                not_divides_pred(p)(a) = (not p.divides(a))
                not_divides_pred(p)(a)
                divisor_list(n).filter(nat_mobius_support).contains(a) and not_divides_pred(p)(a)
                filter_contains_and(divisor_list(n), nat_mobius_support, a)
                fs.contains(a)
                filter_contains_and(fs, not_divides_pred(p), a)
                la.contains(a)
                map_contains_of_contains(la, mul_by_nat(p), a)
                map(la, mul_by_nat(p)).contains(mul_by_nat(p)(a))
                mul_by_nat(p)(a) = p * a
                map(la, mul_by_nat(p)).contains(p * a)
                map(la, mul_by_nat(p)).contains(y)
            }
            map(la, mul_by_nat(p)).contains(y) = lb.contains(y)
        }
        unique_same_contains_imp_permutation(map(la, mul_by_nat(p)), lb)
        is_permutation(map(la, mul_by_nat(p)), lb)
        is_permutation(
            map(divisor_list(n).filter(nat_mobius_support).filter(not_divides_pred(p)),
                mul_by_nat(p)),
            divisor_list(n).filter(nat_mobius_support).filter(divides_pred(p)))
    }
}

/// The sum of the Möbius values over the divisors of `n` vanishes when `n`
/// has a prime divisor `p`.
theorem nat_mobius_pair_sum(n: Nat, p: Nat) {
    Nat.0 < n and p.is_prime and p.divides(n) implies
        sum(map(divisor_list(n), nat_mobius)) = Int.0
} by {
    if Nat.0 < n and p.is_prime and p.divides(n) {
        let fs: List[Nat] = divisor_list(n).filter(nat_mobius_support)
        let la: List[Nat] = fs.filter(not_divides_pred(p))
        let lb: List[Nat] = fs.filter(divides_pred(p))
        forall(x: Nat) {
            if divisor_list(n).contains(x) and not nat_mobius_support(x) {
                not nat_mobius_support(x)
                nat_mobius_support(x) = (nat_mobius(x) != Int.0)
                not (nat_mobius(x) != Int.0)
                nat_mobius(x) = Int.0
            }
            divisor_list(n).contains(x) and not nat_mobius_support(x) implies nat_mobius(x) = Int.0
        }
        if forall(x: Nat) { divisor_list(n).contains(x) and not nat_mobius_support(x) implies nat_mobius(x) = Int.0 } {
            sum_filter_zero_removed(divisor_list(n), nat_mobius_support, nat_mobius)
            sum(map(divisor_list(n), nat_mobius)) = sum(map(fs, nat_mobius))
        }
        sum_split_by_p(fs, p, nat_mobius)
        sum(map(fs, nat_mobius)) =
            sum(map(fs.filter(not_divides_pred(p)), nat_mobius)) +
            sum(map(fs.filter(divides_pred(p)), nat_mobius))
        sum(map(fs, nat_mobius)) =
            sum(map(la, nat_mobius)) + sum(map(lb, nat_mobius))
        if Nat.0 < n and p.is_prime and p.divides(n) {
            mobius_pair_image_permutation(n, p)
            is_permutation(
                map(divisor_list(n).filter(nat_mobius_support).filter(not_divides_pred(p)),
                    mul_by_nat(p)),
                divisor_list(n).filter(nat_mobius_support).filter(divides_pred(p)))
            is_permutation(map(la, mul_by_nat(p)), lb)
        }
        unique_same_contains_map_sum_eq(map(la, mul_by_nat(p)), lb, nat_mobius)
        sum(map(map(la, mul_by_nat(p)), nat_mobius)) = sum(map(lb, nat_mobius))
        map_map(la, mul_by_nat(p), nat_mobius)
        map(map(la, mul_by_nat(p)), nat_mobius) =
            map(la, compose(nat_mobius, mul_by_nat(p)))
        compose(nat_mobius, mul_by_nat(p)) =
            function(x: Nat) { nat_mobius(mul_by_nat(p)(x)) }
        map(la, compose(nat_mobius, mul_by_nat(p))) =
            map(la, function(x: Nat) { nat_mobius(mul_by_nat(p)(x)) })
        map(map(la, mul_by_nat(p)), nat_mobius) =
            map(la, function(x: Nat) { nat_mobius(mul_by_nat(p)(x)) })
        sum(map(map(la, mul_by_nat(p)), nat_mobius)) =
            sum(map(la, function(x: Nat) { nat_mobius(mul_by_nat(p)(x)) }))
        forall(x: Nat) {
            if la.contains(x) {
                fs.filter(not_divides_pred(p)).contains(x)
                filter_contained_by_and(fs, not_divides_pred(p), x)
                fs.contains(x) and not_divides_pred(p)(x)
                filter_contained_by_and(divisor_list(n), nat_mobius_support, x)
                divisor_list(n).filter(nat_mobius_support).contains(x)
                mobius_pair_forward(n, p, x)
                nat_mobius(p * x) = -nat_mobius(x)
                mul_by_nat(p)(x) = p * x
                nat_mobius(mul_by_nat(p)(x)) = -nat_mobius(x)
            }
            la.contains(x) implies nat_mobius(mul_by_nat(p)(x)) = -nat_mobius(x)
        }
        sum_map_of_pointwise(la,
            function(x: Nat) { nat_mobius(mul_by_nat(p)(x)) },
            function(x: Nat) { -nat_mobius(x) })
        sum(map(la, function(x: Nat) { nat_mobius(mul_by_nat(p)(x)) })) =
            sum(map(la, function(x: Nat) { -nat_mobius(x) }))
        int_sum_map_neg(la, nat_mobius)
        sum(map(la, function(x: Nat) { -nat_mobius(x) })) = -sum(map(la, nat_mobius))
        sum(map(la, nat_mobius)) + sum(map(lb, nat_mobius)) =
            sum(map(la, nat_mobius)) + -sum(map(la, nat_mobius))
        add_neg(sum(map(la, nat_mobius)))
        sum(map(la, nat_mobius)) + -sum(map(la, nat_mobius)) = Int.0
        sum(map(la, nat_mobius)) + sum(map(lb, nat_mobius)) = Int.0
        sum(map(fs, nat_mobius)) = Int.0
        sum(map(divisor_list(n), nat_mobius)) = sum(map(fs, nat_mobius))
        sum(map(divisor_list(n), nat_mobius)) = Int.0
    }
}

/// The pairing sum, with the hypothesis split into implications.
theorem nat_mobius_pair_sum_imp(n: Nat, p: Nat) {
    (Nat.0 < n) implies (p.is_prime implies (p.divides(n) implies
        sum(map(divisor_list(n), nat_mobius)) = Int.0))
} by {
    if Nat.0 < n {
        if p.is_prime {
            if p.divides(n) {
                Nat.0 < n and p.is_prime and p.divides(n)
                if Nat.0 < n and p.is_prime and p.divides(n) {
                    nat_mobius_pair_sum(n, p)
                    sum(map(divisor_list(n), nat_mobius)) = Int.0
                }
            }
        }
    }
}

/// The divisor sum of the Möbius function at one is one.
theorem nat_mobius_divisor_sum_one {
    sum(map(divisor_list(Nat.1), nat_mobius)) = Int.1
} by {
    divisor_list_one
    divisor_list(Nat.1) = List.cons(Nat.1, List.nil[Nat])
    nat_mobius_one
    nat_mobius(Nat.1) = Int.1
    map(List.cons(Nat.1, List.nil[Nat]), nat_mobius) =
        List.cons(nat_mobius(Nat.1), map(List.nil[Nat], nat_mobius))
    map(List.nil[Nat], nat_mobius) = List.nil[Int]
    map(List.cons(Nat.1, List.nil[Nat]), nat_mobius) =
        List.cons(Int.1, List.nil[Int])
    sum(List.cons(Int.1, List.nil[Int])) = Int.1 + sum(List.nil[Int])
    sum(List.nil[Int]) = Int.0
    Int.1 + Int.0 = Int.1
    sum(map(divisor_list(Nat.1), nat_mobius)) = Int.1
}

/// A natural other than zero and one is at least two.
theorem nat_lt_one_of_nonzero_nonone(n: Nat) {
    n != Nat.0 and n != Nat.1 implies Nat.1 < n
} by {
    if n != Nat.0 and n != Nat.1 {
        Nat.0 < n
        lt_imp_lte_suc(Nat.0, n)
        Nat.1 <= n
        Nat.1 < n
    }
}

/// The divisor sum of the Möbius function vanishes at zero.
theorem nat_mobius_divisor_sum_at_zero {
    sum(map(divisor_list(Nat.0), nat_mobius)) = Int.0
} by {
    divisor_list_zero
    divisor_list(Nat.0) = List.nil[Nat]
    map(List.nil[Nat], nat_mobius) = List.nil[Int]
    sum(List.nil[Int]) = Int.0
    sum(map(divisor_list(Nat.0), nat_mobius)) = Int.0
}

/// A natural above one is positive.
theorem nat_zero_lt_of_lt_one(n: Nat) {
    Nat.1 < n implies Nat.0 < n
} by {
    if Nat.1 < n {
        Nat.1 < n = (Nat.1 <= n and Nat.1 != n)
        Nat.1 <= n and Nat.1 != n
        Nat.1 <= n
        lt_suc(Nat.0)
        Nat.0 < Nat.1
        lt_and_lte(Nat.0, Nat.1, n)
        Nat.0 < n
    }
}

/// The divisor sum of the Möbius function vanishes above one.
theorem nat_mobius_divisor_sum_gt_one(n: Nat) {
    Nat.1 < n implies sum(map(divisor_list(n), nat_mobius)) = Int.0
} by {
    if Nat.1 < n {
        nat_zero_lt_of_lt_one(n)
        Nat.0 < n
        has_prime_divisor(n)
        let p: Nat satisfy { p.is_prime and p.divides(n) }
        p.is_prime
        p.divides(n)
        nat_mobius_pair_sum_imp(n, p)
        (Nat.0 < n) implies (p.is_prime implies (p.divides(n) implies
            sum(map(divisor_list(n), nat_mobius)) = Int.0))
        Nat.0 < n
        p.is_prime
        p.divides(n)
        sum(map(divisor_list(n), nat_mobius)) = Int.0
    }
}

/// The divisor sum of the Möbius function vanishes away from one.
theorem nat_mobius_divisor_sum_off_one(n: Nat) {
    n != Nat.1 implies sum(map(divisor_list(n), nat_mobius)) = Int.0
} by {
    if n != Nat.1 {
        if n = Nat.0 {
            nat_mobius_divisor_sum_at_zero
            sum(map(divisor_list(n), nat_mobius)) = Int.0
        } else {
            n != Nat.0
            n != Nat.1
            n != Nat.0 and n != Nat.1
            nat_lt_one_of_nonzero_nonone(n)
            Nat.1 < n
            nat_mobius_divisor_sum_gt_one(n)
            sum(map(divisor_list(n), nat_mobius)) = Int.0
        }
    }
}

/// The fundamental identity of the Möbius function:
/// `sum_{d | n} mu(d)` is 1 at `n = 1` and 0 elsewhere.
theorem nat_mobius_divisor_sum(n: Nat) {
    sum(map(divisor_list(n), nat_mobius)) = if n = Nat.1 { Int.1 } else { Int.0 }
} by {
    if n = Nat.1 {
        nat_mobius_divisor_sum_one
        sum(map(divisor_list(n), nat_mobius)) = Int.1
        sum(map(divisor_list(n), nat_mobius)) = if n = Nat.1 { Int.1 } else { Int.0 }
    } else {
        n != Nat.1
        nat_mobius_divisor_sum_off_one(n)
        sum(map(divisor_list(n), nat_mobius)) = Int.0
        sum(map(divisor_list(n), nat_mobius)) = if n = Nat.1 { Int.1 } else { Int.0 }
    }
}

/// The Mertens function `M(n) = sum_{k <= n} mu(k)`.
define mertens(n: Nat) -> Int {
    sum(map(n.suc.range, nat_mobius))
}

/// `M(0) = mu(0) = 0`.
theorem mertens_zero {
    mertens(Nat.0) = Int.0
} by {
    mertens(Nat.0) = sum(map(Nat.0.suc.range, nat_mobius))
    Nat.0.suc = Nat.1
    Nat.1.range = List.nil[Nat] + List.singleton(Nat.0)
    map(Nat.0.suc.range, nat_mobius) = map(List.cons(Nat.0, List.nil[Nat]), nat_mobius)
    map(List.cons(Nat.0, List.nil[Nat]), nat_mobius) =
        List.cons(nat_mobius(Nat.0), map(List.nil[Nat], nat_mobius))
    map(List.nil[Nat], nat_mobius) = List.nil[Int]
    nat_mobius_zero
    nat_mobius(Nat.0) = Int.0
    sum(List.cons(Int.0, List.nil[Int])) = Int.0 + sum(List.nil[Int])
    sum(List.nil[Int]) = Int.0
    Int.0 + Int.0 = Int.0
    mertens(Nat.0) = Int.0
}

/// `M(1) = mu(0) + mu(1) = 1`.
theorem mertens_one {
    mertens(Nat.1) = Int.1
} by {
    mertens(Nat.1) = sum(map(Nat.1.suc.range, nat_mobius))
    Nat.1.suc = Nat.2
    Nat.0.range = List.nil[Nat]
    Nat.1.range = Nat.0.range.append(Nat.0)
    Nat.1.range = List.cons(Nat.0, List.nil[Nat])
    Nat.2.range = Nat.1.range.append(Nat.1)
    List.cons(Nat.0, List.nil[Nat]).append(Nat.1) =
        List.cons(Nat.0, List.cons(Nat.1, List.nil[Nat]))
    Nat.2.range = List.cons(Nat.0, List.cons(Nat.1, List.nil[Nat]))
    map(List.cons(Nat.0, List.cons(Nat.1, List.nil[Nat])), nat_mobius) =
        List.cons(nat_mobius(Nat.0), map(List.cons(Nat.1, List.nil[Nat]), nat_mobius))
    map(List.cons(Nat.1, List.nil[Nat]), nat_mobius) =
        List.cons(nat_mobius(Nat.1), map(List.nil[Nat], nat_mobius))
    map(List.nil[Nat], nat_mobius) = List.nil[Int]
    nat_mobius_zero
    nat_mobius(Nat.0) = Int.0
    nat_mobius_one
    nat_mobius(Nat.1) = Int.1
    sum(List.cons(Int.0, List.cons(Int.1, List.nil[Int]))) =
        Int.0 + sum(List.cons(Int.1, List.nil[Int]))
    sum(List.cons(Int.1, List.nil[Int])) = Int.1 + sum(List.nil[Int])
    sum(List.nil[Int]) = Int.0
    Int.1 + Int.0 = Int.1
    Int.0 + Int.1 = Int.1
    mertens(Nat.1) = Int.1
}

/// Möbius inversion: if `g(n) = sum_{d | n} f(d)`, then
/// `f(n) = sum_{d | n} mu(d) * g(n / d)`.
///
/// The proof expands the inner divisor sum and interchanges the two sums over
/// the divisor pairs `(d, e)` with `d * e | n`, then applies the fundamental
/// identity.  The interchange is the missing piece; the statement is recorded
/// here for the research target.
///
// theorem mobius_inversion(f: Nat -> Int, n: Nat) {
//     Nat.0 < n implies
//         sum(map(divisor_list(n), function(d: Nat) {
//             nat_mobius(d) * sum(map(divisor_list(divisor_quotient(n, d)), f))
//         })) = f(n)
// }
