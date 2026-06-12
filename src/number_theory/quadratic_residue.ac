from number_theory.multiplicative_order import Nat, multiplicative_order_mod,
    multiplicative_order_mod_pow_congr_one, multiplicative_order_mod_divides_exponent,
    multiplicative_order_mod_divides_imp_pow_congr_one
from number_theory.congruence import congr_mod_refl, congr_mod_symm, congr_mod_trans,
    congr_mod_mul, congr_mod_pow
from nat import sq_eq_mul, exp_mul
from number_theory.arithmetic_functions import nat_mul_swap_middle
from number_theory.coprime import coprime_comm, coprime_mul, coprime_mul_imp_left,
    coprime_mod_iff, coprime_pow_right
from number_theory.totient import not_coprime_imp_divides_prime,
    divides_prime_imp_not_coprime
from number_theory.fermat import divides_imp_congr_zero, fermat_euler, prime_divides_mul
from nat import add_imp_sub, div_sub_mod, mod_of_zero, sub_zero
numerals Nat

/// True when `a` is a square modulo `n`.
define is_quadratic_residue_mod(a: Nat, n: Nat) -> Bool {
    exists(x: Nat) { x.pow(Nat.2).congr_mod(a, n) }
}

/// True when `a` is represented by the square of a unit modulo `n`.
define is_unit_quadratic_residue_mod(a: Nat, n: Nat) -> Bool {
    exists(x: Nat) { x.coprime(n) and x.pow(Nat.2).congr_mod(a, n) }
}

/// A unit quadratic residue is, in particular, a quadratic residue.
theorem unit_quadratic_residue_is_residue(a: Nat, n: Nat) {
    is_unit_quadratic_residue_mod(a, n) implies is_quadratic_residue_mod(a, n)
} by {
    if is_unit_quadratic_residue_mod(a, n) {
        let x: Nat satisfy { x.coprime(n) and x.pow(Nat.2).congr_mod(a, n) }
        exists(y: Nat) { y.pow(Nat.2).congr_mod(a, n) }
    }
}

/// A square is a quadratic residue modulo every modulus.
theorem square_is_quadratic_residue_mod(x: Nat, n: Nat) {
    is_quadratic_residue_mod(x.pow(Nat.2), n)
} by {
    congr_mod_refl(x.pow(Nat.2), n)
    exists(y: Nat) { y.pow(Nat.2).congr_mod(x.pow(Nat.2), n) }
}

/// The square of a unit is a unit quadratic residue.
theorem unit_square_is_unit_quadratic_residue_mod(x: Nat, n: Nat) {
    x.coprime(n) implies is_unit_quadratic_residue_mod(x.pow(Nat.2), n)
} by {
    if x.coprime(n) {
        congr_mod_refl(x.pow(Nat.2), n)
        exists(y: Nat) { y.coprime(n) and y.pow(Nat.2).congr_mod(x.pow(Nat.2), n) }
    }
}

/// The zero class is a quadratic residue modulo every modulus.
theorem zero_quadratic_residue_mod(n: Nat) {
    is_quadratic_residue_mod(Nat.0, n)
} by {
    sq_eq_mul(Nat.0)
    Nat.0.pow(Nat.2) = Nat.0 * Nat.0
    Nat.0 * Nat.0 = Nat.0
    Nat.0.pow(Nat.2) = Nat.0
    congr_mod_refl(Nat.0, n)
    Nat.0.pow(Nat.2).congr_mod(Nat.0, n)
    exists(x: Nat) { x.pow(Nat.2).congr_mod(Nat.0, n) }
}

/// A number congruent to an explicit square is a quadratic residue.
theorem quadratic_residue_of_square_congr(x: Nat, a: Nat, n: Nat) {
    x.pow(Nat.2).congr_mod(a, n) implies is_quadratic_residue_mod(a, n)
} by {
    if x.pow(Nat.2).congr_mod(a, n) {
        exists(y: Nat) { y.pow(Nat.2).congr_mod(a, n) }
    }
}

/// A number congruent to the square of a unit is a unit quadratic residue.
theorem unit_quadratic_residue_of_unit_square_congr(x: Nat, a: Nat, n: Nat) {
    x.coprime(n) and x.pow(Nat.2).congr_mod(a, n)
        implies is_unit_quadratic_residue_mod(a, n)
} by {
    if x.coprime(n) and x.pow(Nat.2).congr_mod(a, n) {
        exists(y: Nat) { y.coprime(n) and y.pow(Nat.2).congr_mod(a, n) }
    }
}

/// Congruent roots have congruent squares.
theorem square_congr_mod(x: Nat, y: Nat, n: Nat) {
    x.congr_mod(y, n) implies x.pow(Nat.2).congr_mod(y.pow(Nat.2), n)
} by {
    if x.congr_mod(y, n) {
        congr_mod_pow(x, y, n, Nat.2)
    }
}

/// Replacing a square root by a congruent root preserves its represented
/// quadratic residue.
theorem quadratic_residue_of_congruent_root(x: Nat, y: Nat, a: Nat, n: Nat) {
    x.congr_mod(y, n) and y.pow(Nat.2).congr_mod(a, n)
        implies is_quadratic_residue_mod(a, n)
} by {
    if x.congr_mod(y, n) and y.pow(Nat.2).congr_mod(a, n) {
        square_congr_mod(x, y, n)
        x.pow(Nat.2).congr_mod(y.pow(Nat.2), n)
        congr_mod_trans(x.pow(Nat.2), y.pow(Nat.2), a, n)
        x.pow(Nat.2).congr_mod(a, n)
        quadratic_residue_of_square_congr(x, a, n)
    }
}

/// Replacing a unit square root by a congruent unit root preserves its unit
/// quadratic residue.
theorem unit_quadratic_residue_of_congruent_unit_root(
    x: Nat, y: Nat, a: Nat, n: Nat
) {
    x.coprime(n) and x.congr_mod(y, n) and y.pow(Nat.2).congr_mod(a, n)
        implies is_unit_quadratic_residue_mod(a, n)
} by {
    if x.coprime(n) and x.congr_mod(y, n) and y.pow(Nat.2).congr_mod(a, n) {
        square_congr_mod(x, y, n)
        x.pow(Nat.2).congr_mod(y.pow(Nat.2), n)
        congr_mod_trans(x.pow(Nat.2), y.pow(Nat.2), a, n)
        x.pow(Nat.2).congr_mod(a, n)
        unit_quadratic_residue_of_unit_square_congr(x, a, n)
        is_unit_quadratic_residue_mod(a, n)
    }
}

/// Congruent targets have the same quadratic-residue status.
theorem quadratic_residue_congr_mod(a: Nat, b: Nat, n: Nat) {
    is_quadratic_residue_mod(a, n) and a.congr_mod(b, n)
        implies is_quadratic_residue_mod(b, n)
} by {
    if is_quadratic_residue_mod(a, n) and a.congr_mod(b, n) {
        let x: Nat satisfy { x.pow(Nat.2).congr_mod(a, n) }
        congr_mod_trans(x.pow(Nat.2), a, b, n)
        x.pow(Nat.2).congr_mod(b, n)
        exists(y: Nat) { y.pow(Nat.2).congr_mod(b, n) }
    }
}

/// Congruent targets have the same unit-quadratic-residue status.
theorem unit_quadratic_residue_congr_mod(a: Nat, b: Nat, n: Nat) {
    is_unit_quadratic_residue_mod(a, n) and a.congr_mod(b, n)
        implies is_unit_quadratic_residue_mod(b, n)
} by {
    if is_unit_quadratic_residue_mod(a, n) and a.congr_mod(b, n) {
        let x: Nat satisfy { x.coprime(n) and x.pow(Nat.2).congr_mod(a, n) }
        congr_mod_trans(x.pow(Nat.2), a, b, n)
        x.pow(Nat.2).congr_mod(b, n)
        exists(y: Nat) { y.coprime(n) and y.pow(Nat.2).congr_mod(b, n) }
    }
}

/// Congruent targets have equivalent quadratic-residue status.
theorem quadratic_residue_congr_mod_iff(a: Nat, b: Nat, n: Nat) {
    a.congr_mod(b, n)
        implies is_quadratic_residue_mod(a, n) = is_quadratic_residue_mod(b, n)
} by {
    if a.congr_mod(b, n) {
        if is_quadratic_residue_mod(a, n) {
            quadratic_residue_congr_mod(a, b, n)
            is_quadratic_residue_mod(b, n)
        }
        if is_quadratic_residue_mod(b, n) {
            congr_mod_symm(a, b, n)
            b.congr_mod(a, n)
            quadratic_residue_congr_mod(b, a, n)
            is_quadratic_residue_mod(a, n)
        }
        (is_quadratic_residue_mod(a, n) = is_quadratic_residue_mod(b, n)) = true
    }
}

/// Congruent targets have equivalent unit-quadratic-residue status.
theorem unit_quadratic_residue_congr_mod_iff(a: Nat, b: Nat, n: Nat) {
    a.congr_mod(b, n)
        implies is_unit_quadratic_residue_mod(a, n) = is_unit_quadratic_residue_mod(b, n)
} by {
    if a.congr_mod(b, n) {
        if is_unit_quadratic_residue_mod(a, n) {
            unit_quadratic_residue_congr_mod(a, b, n)
            is_unit_quadratic_residue_mod(b, n)
        }
        if is_unit_quadratic_residue_mod(b, n) {
            congr_mod_symm(a, b, n)
            b.congr_mod(a, n)
            unit_quadratic_residue_congr_mod(b, a, n)
            is_unit_quadratic_residue_mod(a, n)
        }
        (is_unit_quadratic_residue_mod(a, n) =
            is_unit_quadratic_residue_mod(b, n)) = true
    }
}

/// The square of a product is the product of the squares.
theorem square_mul_eq_mul_squares(x: Nat, y: Nat) {
    (x * y).pow(Nat.2) = x.pow(Nat.2) * y.pow(Nat.2)
} by {
    sq_eq_mul(x * y)
    (x * y).pow(Nat.2) = (x * y) * (x * y)
    sq_eq_mul(x)
    x.pow(Nat.2) = x * x
    sq_eq_mul(y)
    y.pow(Nat.2) = y * y
    x.pow(Nat.2) * y.pow(Nat.2) = (x * x) * (y * y)
    nat_mul_swap_middle(x, y, x, y)
    (x * y) * (x * y) = (x * x) * (y * y)
    (x * y).pow(Nat.2) = x.pow(Nat.2) * y.pow(Nat.2)
}

/// Quadratic residues are closed under multiplication.
theorem quadratic_residue_mul(a: Nat, b: Nat, n: Nat) {
    is_quadratic_residue_mod(a, n) and is_quadratic_residue_mod(b, n)
        implies is_quadratic_residue_mod(a * b, n)
} by {
    if is_quadratic_residue_mod(a, n) and is_quadratic_residue_mod(b, n) {
        let x: Nat satisfy { x.pow(Nat.2).congr_mod(a, n) }
        let y: Nat satisfy { y.pow(Nat.2).congr_mod(b, n) }
        congr_mod_mul(x.pow(Nat.2), y.pow(Nat.2), a, b, n)
        (x.pow(Nat.2) * y.pow(Nat.2)).congr_mod(a * b, n)
        square_mul_eq_mul_squares(x, y)
        (x * y).pow(Nat.2) = x.pow(Nat.2) * y.pow(Nat.2)
        (x * y).pow(Nat.2).congr_mod(a * b, n)
        exists(z: Nat) { z.pow(Nat.2).congr_mod(a * b, n) }
    }
}

/// A target congruent to a product of quadratic residues is a quadratic
/// residue.
theorem quadratic_residue_mul_congr(a: Nat, b: Nat, c: Nat, n: Nat) {
    is_quadratic_residue_mod(a, n) and is_quadratic_residue_mod(b, n)
        and (a * b).congr_mod(c, n)
        implies is_quadratic_residue_mod(c, n)
} by {
    if is_quadratic_residue_mod(a, n) and is_quadratic_residue_mod(b, n)
        and (a * b).congr_mod(c, n) {
        quadratic_residue_mul(a, b, n)
        is_quadratic_residue_mod(a * b, n)
        quadratic_residue_congr_mod(a * b, c, n)
    }
}

/// The square of a quadratic residue is a quadratic residue.
theorem quadratic_residue_square(a: Nat, n: Nat) {
    is_quadratic_residue_mod(a, n) implies is_quadratic_residue_mod(a.pow(Nat.2), n)
} by {
    if is_quadratic_residue_mod(a, n) {
        quadratic_residue_mul(a, a, n)
        is_quadratic_residue_mod(a * a, n)
        sq_eq_mul(a)
        a.pow(Nat.2) = a * a
        is_quadratic_residue_mod(a.pow(Nat.2), n)
    }
}

/// A target congruent to the square of a quadratic residue is a quadratic
/// residue.
theorem quadratic_residue_square_congr(a: Nat, b: Nat, n: Nat) {
    is_quadratic_residue_mod(a, n) and a.pow(Nat.2).congr_mod(b, n)
        implies is_quadratic_residue_mod(b, n)
} by {
    if is_quadratic_residue_mod(a, n) and a.pow(Nat.2).congr_mod(b, n) {
        quadratic_residue_square(a, n)
        is_quadratic_residue_mod(a.pow(Nat.2), n)
        quadratic_residue_congr_mod(a.pow(Nat.2), b, n)
        is_quadratic_residue_mod(b, n)
    }
}

/// Any power of a quadratic residue is a quadratic residue.
theorem quadratic_residue_power(a: Nat, n: Nat, m: Nat) {
    is_quadratic_residue_mod(a, n) implies is_quadratic_residue_mod(a.pow(m), n)
} by {
    if is_quadratic_residue_mod(a, n) {
        let x: Nat satisfy { x.pow(Nat.2).congr_mod(a, n) }
        congr_mod_pow(x.pow(Nat.2), a, n, m)
        x.pow(Nat.2).pow(m).congr_mod(a.pow(m), n)
        exp_mul(x, Nat.2, m)
        x.pow(Nat.2 * m) = x.pow(Nat.2).pow(m)
        exp_mul(x, m, Nat.2)
        x.pow(m * Nat.2) = x.pow(m).pow(Nat.2)
        Nat.2 * m = m * Nat.2
        x.pow(Nat.2 * m) = x.pow(m * Nat.2)
        x.pow(Nat.2).pow(m) = x.pow(m).pow(Nat.2)
        x.pow(m).pow(Nat.2).congr_mod(a.pow(m), n)
        quadratic_residue_of_square_congr(x.pow(m), a.pow(m), n)
        is_quadratic_residue_mod(a.pow(m), n)
    }
}

/// A target congruent to a power of a quadratic residue is a quadratic
/// residue.
theorem quadratic_residue_power_congr(a: Nat, b: Nat, n: Nat, m: Nat) {
    is_quadratic_residue_mod(a, n) and a.pow(m).congr_mod(b, n)
        implies is_quadratic_residue_mod(b, n)
} by {
    if is_quadratic_residue_mod(a, n) and a.pow(m).congr_mod(b, n) {
        quadratic_residue_power(a, n, m)
        is_quadratic_residue_mod(a.pow(m), n)
        quadratic_residue_congr_mod(a.pow(m), b, n)
        is_quadratic_residue_mod(b, n)
    }
}

/// Multiplying a quadratic residue by a square gives a quadratic residue.
theorem quadratic_residue_mul_square(a: Nat, x: Nat, n: Nat) {
    is_quadratic_residue_mod(a, n) implies
        is_quadratic_residue_mod(a * x.pow(Nat.2), n)
} by {
    if is_quadratic_residue_mod(a, n) {
        square_is_quadratic_residue_mod(x, n)
        is_quadratic_residue_mod(x.pow(Nat.2), n)
        quadratic_residue_mul(a, x.pow(Nat.2), n)
    }
}

/// Multiplying a square by a quadratic residue gives a quadratic residue.
theorem quadratic_residue_square_mul(x: Nat, a: Nat, n: Nat) {
    is_quadratic_residue_mod(a, n) implies
        is_quadratic_residue_mod(x.pow(Nat.2) * a, n)
} by {
    if is_quadratic_residue_mod(a, n) {
        square_is_quadratic_residue_mod(x, n)
        is_quadratic_residue_mod(x.pow(Nat.2), n)
        quadratic_residue_mul(x.pow(Nat.2), a, n)
    }
}

/// A target congruent to a square times a quadratic residue is a quadratic
/// residue.
theorem quadratic_residue_square_mul_congr(x: Nat, a: Nat, c: Nat, n: Nat) {
    is_quadratic_residue_mod(a, n) and (x.pow(Nat.2) * a).congr_mod(c, n)
        implies is_quadratic_residue_mod(c, n)
} by {
    if is_quadratic_residue_mod(a, n) and (x.pow(Nat.2) * a).congr_mod(c, n) {
        quadratic_residue_square_mul(x, a, n)
        is_quadratic_residue_mod(x.pow(Nat.2) * a, n)
        quadratic_residue_congr_mod(x.pow(Nat.2) * a, c, n)
        is_quadratic_residue_mod(c, n)
    }
}

/// If a number is coprime to `n`, then so is its square.
theorem square_coprime(x: Nat, n: Nat) {
    x.coprime(n) implies x.pow(Nat.2).coprime(n)
} by {
    if x.coprime(n) {
        coprime_comm(x, n)
        n.coprime(x)
        coprime_mul(n, x, x)
        n.coprime(x * x)
        coprime_comm(n, x * x)
        (x * x).coprime(n)
        sq_eq_mul(x)
        x.pow(Nat.2) = x * x
        x.pow(Nat.2).coprime(n)
    }
}

/// If a square is coprime to `n`, then its base is coprime to `n`.
theorem square_coprime_imp_base(x: Nat, n: Nat) {
    x.pow(Nat.2).coprime(n) implies x.coprime(n)
} by {
    if x.pow(Nat.2).coprime(n) {
        sq_eq_mul(x)
        x.pow(Nat.2) = x * x
        (x * x).coprime(n)
        coprime_comm(x * x, n)
        n.coprime(x * x)
        coprime_mul_imp_left(n, x, x)
        n.coprime(x)
        coprime_comm(n, x)
        x.coprime(n)
    }
}

/// Coprimality with the modulus is invariant under congruence.
theorem congr_mod_preserves_coprime(a: Nat, b: Nat, n: Nat) {
    a.congr_mod(b, n) and a.coprime(n) implies b.coprime(n)
} by {
    if a.congr_mod(b, n) and a.coprime(n) {
        a.mod(n) = b.mod(n)
        coprime_mod_iff(a, n)
        a.coprime(n) = a.mod(n).coprime(n)
        a.mod(n).coprime(n)
        b.mod(n).coprime(n)
        coprime_mod_iff(b, n)
        b.coprime(n) = b.mod(n).coprime(n)
        b.coprime(n)
    }
}

/// A coprime quadratic residue is represented by a square root that is a unit.
theorem quadratic_residue_coprime_is_unit(a: Nat, n: Nat) {
    is_quadratic_residue_mod(a, n) and a.coprime(n)
        implies is_unit_quadratic_residue_mod(a, n)
} by {
    if is_quadratic_residue_mod(a, n) and a.coprime(n) {
        let x: Nat satisfy { x.pow(Nat.2).congr_mod(a, n) }
        congr_mod_symm(x.pow(Nat.2), a, n)
        a.congr_mod(x.pow(Nat.2), n)
        congr_mod_preserves_coprime(a, x.pow(Nat.2), n)
        x.pow(Nat.2).coprime(n)
        square_coprime_imp_base(x, n)
        x.coprime(n)
        exists(y: Nat) { y.coprime(n) and y.pow(Nat.2).congr_mod(a, n) }
    }
}

/// On coprime targets, ordinary and unit quadratic-residue predicates agree.
theorem quadratic_residue_coprime_iff_unit(a: Nat, n: Nat) {
    a.coprime(n) implies
        is_quadratic_residue_mod(a, n) = is_unit_quadratic_residue_mod(a, n)
} by {
    if a.coprime(n) {
        if is_quadratic_residue_mod(a, n) {
            quadratic_residue_coprime_is_unit(a, n)
            is_unit_quadratic_residue_mod(a, n)
        }
        if is_unit_quadratic_residue_mod(a, n) {
            unit_quadratic_residue_is_residue(a, n)
            is_quadratic_residue_mod(a, n)
        }
        (is_quadratic_residue_mod(a, n) = is_unit_quadratic_residue_mod(a, n)) = true
    }
}

/// Unit quadratic residues are coprime to the modulus.
theorem unit_quadratic_residue_coprime(a: Nat, n: Nat) {
    is_unit_quadratic_residue_mod(a, n) implies a.coprime(n)
} by {
    if is_unit_quadratic_residue_mod(a, n) {
        let x: Nat satisfy { x.coprime(n) and x.pow(Nat.2).congr_mod(a, n) }
        square_coprime(x, n)
        x.pow(Nat.2).coprime(n)
        congr_mod_preserves_coprime(x.pow(Nat.2), a, n)
        a.coprime(n)
    }
}

/// Unit quadratic residues are exactly quadratic residues that are coprime to
/// the modulus.
theorem unit_quadratic_residue_iff_residue_and_coprime(a: Nat, n: Nat) {
    is_unit_quadratic_residue_mod(a, n) =
        (is_quadratic_residue_mod(a, n) and a.coprime(n))
} by {
    if is_unit_quadratic_residue_mod(a, n) {
        unit_quadratic_residue_is_residue(a, n)
        is_quadratic_residue_mod(a, n)
        unit_quadratic_residue_coprime(a, n)
        a.coprime(n)
        is_quadratic_residue_mod(a, n) and a.coprime(n)
    }
    if is_quadratic_residue_mod(a, n) and a.coprime(n) {
        quadratic_residue_coprime_is_unit(a, n)
        is_unit_quadratic_residue_mod(a, n)
    }
    (is_unit_quadratic_residue_mod(a, n) =
        (is_quadratic_residue_mod(a, n) and a.coprime(n))) = true
}

/// Unit quadratic residues are closed under multiplication.
theorem unit_quadratic_residue_mul(a: Nat, b: Nat, n: Nat) {
    is_unit_quadratic_residue_mod(a, n) and is_unit_quadratic_residue_mod(b, n)
        implies is_unit_quadratic_residue_mod(a * b, n)
} by {
    if is_unit_quadratic_residue_mod(a, n) and is_unit_quadratic_residue_mod(b, n) {
        let x: Nat satisfy { x.coprime(n) and x.pow(Nat.2).congr_mod(a, n) }
        let y: Nat satisfy { y.coprime(n) and y.pow(Nat.2).congr_mod(b, n) }
        coprime_comm(x, n)
        n.coprime(x)
        coprime_comm(y, n)
        n.coprime(y)
        coprime_mul(n, x, y)
        n.coprime(x * y)
        coprime_comm(n, x * y)
        (x * y).coprime(n)
        congr_mod_mul(x.pow(Nat.2), y.pow(Nat.2), a, b, n)
        (x.pow(Nat.2) * y.pow(Nat.2)).congr_mod(a * b, n)
        square_mul_eq_mul_squares(x, y)
        (x * y).pow(Nat.2) = x.pow(Nat.2) * y.pow(Nat.2)
        (x * y).pow(Nat.2).congr_mod(a * b, n)
        exists(z: Nat) { z.coprime(n) and z.pow(Nat.2).congr_mod(a * b, n) }
    }
}

/// A target congruent to a product of unit quadratic residues is a unit
/// quadratic residue.
theorem unit_quadratic_residue_mul_congr(a: Nat, b: Nat, c: Nat, n: Nat) {
    is_unit_quadratic_residue_mod(a, n) and is_unit_quadratic_residue_mod(b, n)
        and (a * b).congr_mod(c, n)
        implies is_unit_quadratic_residue_mod(c, n)
} by {
    if is_unit_quadratic_residue_mod(a, n) and is_unit_quadratic_residue_mod(b, n)
        and (a * b).congr_mod(c, n) {
        unit_quadratic_residue_mul(a, b, n)
        is_unit_quadratic_residue_mod(a * b, n)
        unit_quadratic_residue_congr_mod(a * b, c, n)
    }
}

/// Multiplying a unit quadratic residue by a unit square gives a unit
/// quadratic residue.
theorem unit_quadratic_residue_mul_unit_square(a: Nat, x: Nat, n: Nat) {
    is_unit_quadratic_residue_mod(a, n) and x.coprime(n)
        implies is_unit_quadratic_residue_mod(a * x.pow(Nat.2), n)
} by {
    if is_unit_quadratic_residue_mod(a, n) and x.coprime(n) {
        unit_square_is_unit_quadratic_residue_mod(x, n)
        is_unit_quadratic_residue_mod(x.pow(Nat.2), n)
        unit_quadratic_residue_mul(a, x.pow(Nat.2), n)
    }
}

/// The square of a unit quadratic residue is a unit quadratic residue.
theorem unit_quadratic_residue_square(a: Nat, n: Nat) {
    is_unit_quadratic_residue_mod(a, n)
        implies is_unit_quadratic_residue_mod(a.pow(Nat.2), n)
} by {
    if is_unit_quadratic_residue_mod(a, n) {
        unit_quadratic_residue_mul(a, a, n)
        is_unit_quadratic_residue_mod(a * a, n)
        sq_eq_mul(a)
        a.pow(Nat.2) = a * a
        is_unit_quadratic_residue_mod(a.pow(Nat.2), n)
    }
}

/// A target congruent to the square of a unit quadratic residue is a unit
/// quadratic residue.
theorem unit_quadratic_residue_square_congr(a: Nat, b: Nat, n: Nat) {
    is_unit_quadratic_residue_mod(a, n) and a.pow(Nat.2).congr_mod(b, n)
        implies is_unit_quadratic_residue_mod(b, n)
} by {
    if is_unit_quadratic_residue_mod(a, n) and a.pow(Nat.2).congr_mod(b, n) {
        unit_quadratic_residue_square(a, n)
        is_unit_quadratic_residue_mod(a.pow(Nat.2), n)
        unit_quadratic_residue_congr_mod(a.pow(Nat.2), b, n)
        is_unit_quadratic_residue_mod(b, n)
    }
}

/// Any power of a unit quadratic residue is a unit quadratic residue.
theorem unit_quadratic_residue_power(a: Nat, n: Nat, m: Nat) {
    is_unit_quadratic_residue_mod(a, n)
        implies is_unit_quadratic_residue_mod(a.pow(m), n)
} by {
    if is_unit_quadratic_residue_mod(a, n) {
        let x: Nat satisfy { x.coprime(n) and x.pow(Nat.2).congr_mod(a, n) }
        congr_mod_pow(x.pow(Nat.2), a, n, m)
        x.pow(Nat.2).pow(m).congr_mod(a.pow(m), n)
        exp_mul(x, Nat.2, m)
        x.pow(Nat.2 * m) = x.pow(Nat.2).pow(m)
        exp_mul(x, m, Nat.2)
        x.pow(m * Nat.2) = x.pow(m).pow(Nat.2)
        Nat.2 * m = m * Nat.2
        x.pow(Nat.2 * m) = x.pow(m * Nat.2)
        x.pow(Nat.2).pow(m) = x.pow(m).pow(Nat.2)
        x.pow(m).pow(Nat.2).congr_mod(a.pow(m), n)
        coprime_pow_right(x, n, m)
        x.pow(m).coprime(n)
        unit_quadratic_residue_of_unit_square_congr(x.pow(m), a.pow(m), n)
        is_unit_quadratic_residue_mod(a.pow(m), n)
    }
}

/// A target congruent to a power of a unit quadratic residue is a unit
/// quadratic residue.
theorem unit_quadratic_residue_power_congr(a: Nat, b: Nat, n: Nat, m: Nat) {
    is_unit_quadratic_residue_mod(a, n) and a.pow(m).congr_mod(b, n)
        implies is_unit_quadratic_residue_mod(b, n)
} by {
    if is_unit_quadratic_residue_mod(a, n) and a.pow(m).congr_mod(b, n) {
        unit_quadratic_residue_power(a, n, m)
        is_unit_quadratic_residue_mod(a.pow(m), n)
        unit_quadratic_residue_congr_mod(a.pow(m), b, n)
        is_unit_quadratic_residue_mod(b, n)
    }
}

/// Multiplying a unit square by a unit quadratic residue gives a unit
/// quadratic residue.
theorem unit_quadratic_residue_unit_square_mul(x: Nat, a: Nat, n: Nat) {
    x.coprime(n) and is_unit_quadratic_residue_mod(a, n)
        implies is_unit_quadratic_residue_mod(x.pow(Nat.2) * a, n)
} by {
    if x.coprime(n) and is_unit_quadratic_residue_mod(a, n) {
        unit_square_is_unit_quadratic_residue_mod(x, n)
        is_unit_quadratic_residue_mod(x.pow(Nat.2), n)
        unit_quadratic_residue_mul(x.pow(Nat.2), a, n)
    }
}

/// A target congruent to a unit square times a unit quadratic residue is a
/// unit quadratic residue.
theorem unit_quadratic_residue_unit_square_mul_congr(
    x: Nat, a: Nat, c: Nat, n: Nat
) {
    x.coprime(n) and is_unit_quadratic_residue_mod(a, n)
        and (x.pow(Nat.2) * a).congr_mod(c, n)
        implies is_unit_quadratic_residue_mod(c, n)
} by {
    if x.coprime(n) and is_unit_quadratic_residue_mod(a, n)
        and (x.pow(Nat.2) * a).congr_mod(c, n) {
        unit_quadratic_residue_unit_square_mul(x, a, n)
        is_unit_quadratic_residue_mod(x.pow(Nat.2) * a, n)
        unit_quadratic_residue_congr_mod(x.pow(Nat.2) * a, c, n)
        is_unit_quadratic_residue_mod(c, n)
    }
}

/// A target congruent to a product of a quadratic residue and a square is a
/// quadratic residue.
theorem quadratic_residue_mul_square_congr(a: Nat, x: Nat, c: Nat, n: Nat) {
    is_quadratic_residue_mod(a, n) and (a * x.pow(Nat.2)).congr_mod(c, n)
        implies is_quadratic_residue_mod(c, n)
} by {
    if is_quadratic_residue_mod(a, n) and (a * x.pow(Nat.2)).congr_mod(c, n) {
        quadratic_residue_mul_square(a, x, n)
        is_quadratic_residue_mod(a * x.pow(Nat.2), n)
        quadratic_residue_congr_mod(a * x.pow(Nat.2), c, n)
    }
}

/// A target congruent to a product of a unit quadratic residue and a unit
/// square is a unit quadratic residue.
theorem unit_quadratic_residue_mul_unit_square_congr(
    a: Nat, x: Nat, c: Nat, n: Nat
) {
    is_unit_quadratic_residue_mod(a, n) and x.coprime(n)
        and (a * x.pow(Nat.2)).congr_mod(c, n)
        implies is_unit_quadratic_residue_mod(c, n)
} by {
    if is_unit_quadratic_residue_mod(a, n) and x.coprime(n)
        and (a * x.pow(Nat.2)).congr_mod(c, n) {
        unit_quadratic_residue_mul_unit_square(a, x, n)
        is_unit_quadratic_residue_mod(a * x.pow(Nat.2), n)
        unit_quadratic_residue_congr_mod(a * x.pow(Nat.2), c, n)
        is_unit_quadratic_residue_mod(c, n)
    }
}

/// An even power of a natural number is a quadratic residue.
theorem even_power_is_quadratic_residue_mod(g: Nat, k: Nat, n: Nat) {
    Nat.2.divides(k) implies is_quadratic_residue_mod(g.pow(k), n)
} by {
    if Nat.2.divides(k) {
        let q: Nat satisfy { Nat.2 * q = k }
        q * Nat.2 = Nat.2 * q
        q * Nat.2 = k
        exp_mul(g, q, Nat.2)
        g.pow(q * Nat.2) = g.pow(q).pow(Nat.2)
        g.pow(k) = g.pow(q).pow(Nat.2)
        g.pow(q).pow(Nat.2) = g.pow(k)
        congr_mod_refl(g.pow(k), n)
        g.pow(q).pow(Nat.2).congr_mod(g.pow(k), n)
        quadratic_residue_of_square_congr(g.pow(q), g.pow(k), n)
        is_quadratic_residue_mod(g.pow(k), n)
    }
}

/// A number congruent to an even power is a quadratic residue.
theorem even_power_congr_quadratic_residue_mod(g: Nat, k: Nat, a: Nat, n: Nat) {
    a.congr_mod(g.pow(k), n) and Nat.2.divides(k)
        implies is_quadratic_residue_mod(a, n)
} by {
    if a.congr_mod(g.pow(k), n) and Nat.2.divides(k) {
        even_power_is_quadratic_residue_mod(g, k, n)
        is_quadratic_residue_mod(g.pow(k), n)
        congr_mod_symm(a, g.pow(k), n)
        g.pow(k).congr_mod(a, n)
        quadratic_residue_congr_mod(g.pow(k), a, n)
        is_quadratic_residue_mod(a, n)
    }
}

/// Congruence to zero modulo `d` is the divisibility relation by `d`.
theorem congr_mod_zero_imp_divides(d: Nat, x: Nat) {
    x.congr_mod(Nat.0, d) implies d.divides(x)
} by {
    if x.congr_mod(Nat.0, d) {
        x.mod(d) = Nat.0.mod(d)
        mod_of_zero(d)
        Nat.0.mod(d) = Nat.0
        x.mod(d) = Nat.0
        div_sub_mod(x, d)
        d.divides(x - x.mod(d))
        x - x.mod(d) = x - Nat.0
        sub_zero(x)
        x - Nat.0 = x
        d.divides(x)
    }
}

/// If a root is congruent to zero, then its square is congruent to zero.
theorem square_congr_zero_of_congr_zero(x: Nat, n: Nat) {
    x.congr_mod(Nat.0, n) implies x.pow(Nat.2).congr_mod(Nat.0, n)
} by {
    if x.congr_mod(Nat.0, n) {
        congr_mod_mul(x, x, Nat.0, Nat.0, n)
        (x * x).congr_mod(Nat.0 * Nat.0, n)
        Nat.0 * Nat.0 = Nat.0
        sq_eq_mul(x)
        x.pow(Nat.2) = x * x
        x.pow(Nat.2).congr_mod(Nat.0, n)
    }
}

/// A square root congruent to zero forces its target residue to be zero.
theorem quadratic_root_zero_imp_target_zero(a: Nat, n: Nat, x: Nat) {
    x.congr_mod(Nat.0, n) and x.pow(Nat.2).congr_mod(a, n)
        implies a.congr_mod(Nat.0, n)
} by {
    if x.congr_mod(Nat.0, n) and x.pow(Nat.2).congr_mod(a, n) {
        square_congr_zero_of_congr_zero(x, n)
        x.pow(Nat.2).congr_mod(Nat.0, n)
        congr_mod_symm(x.pow(Nat.2), a, n)
        a.congr_mod(x.pow(Nat.2), n)
        congr_mod_trans(a, x.pow(Nat.2), Nat.0, n)
        a.congr_mod(Nat.0, n)
    }
}

/// A nonzero represented residue has no zero square root.
theorem quadratic_nonzero_target_imp_nonzero_root(a: Nat, n: Nat, x: Nat) {
    not a.congr_mod(Nat.0, n) and x.pow(Nat.2).congr_mod(a, n)
        implies not x.congr_mod(Nat.0, n)
} by {
    if not a.congr_mod(Nat.0, n) and x.pow(Nat.2).congr_mod(a, n) {
        if x.congr_mod(Nat.0, n) {
            quadratic_root_zero_imp_target_zero(a, n, x)
            a.congr_mod(Nat.0, n)
            false
        }
    }
}

/// A square that is zero modulo a prime has a zero root modulo that prime.
theorem prime_square_congr_zero_imp_root_zero(p: Nat, x: Nat) {
    p.is_prime and x.pow(Nat.2).congr_mod(Nat.0, p)
        implies x.congr_mod(Nat.0, p)
} by {
    if p.is_prime and x.pow(Nat.2).congr_mod(Nat.0, p) {
        congr_mod_zero_imp_divides(p, x.pow(Nat.2))
        p.divides(x.pow(Nat.2))
        sq_eq_mul(x)
        x.pow(Nat.2) = x * x
        p.divides(x * x)
        prime_divides_mul(p, x, x)
        p.divides(x) or p.divides(x)
        p.divides(x)
        divides_imp_congr_zero(p, x)
        x.congr_mod(Nat.0, p)
    }
}

/// A nonzero square root modulo a prime has a nonzero target.
theorem prime_nonzero_root_imp_nonzero_target(p: Nat, a: Nat, x: Nat) {
    p.is_prime and not x.congr_mod(Nat.0, p) and x.pow(Nat.2).congr_mod(a, p)
        implies not a.congr_mod(Nat.0, p)
} by {
    if p.is_prime and not x.congr_mod(Nat.0, p) and x.pow(Nat.2).congr_mod(a, p) {
        if a.congr_mod(Nat.0, p) {
            congr_mod_trans(x.pow(Nat.2), a, Nat.0, p)
            x.pow(Nat.2).congr_mod(Nat.0, p)
            prime_square_congr_zero_imp_root_zero(p, x)
            x.congr_mod(Nat.0, p)
            false
        }
    }
}

/// For a prime modulus, every nonzero residue class is represented by a unit.
theorem prime_nonzero_congr_mod_imp_coprime(p: Nat, x: Nat) {
    p.is_prime and not x.congr_mod(Nat.0, p) implies x.coprime(p)
} by {
    if p.is_prime and not x.congr_mod(Nat.0, p) {
        if not x.coprime(p) {
            not_coprime_imp_divides_prime(p, x)
            p.divides(x)
            divides_imp_congr_zero(p, x)
            x.congr_mod(Nat.0, p)
            false
        }
    }
}

/// For a prime modulus, every coprime residue class is nonzero.
theorem prime_coprime_imp_nonzero_congr_mod(p: Nat, x: Nat) {
    p.is_prime and x.coprime(p) implies not x.congr_mod(Nat.0, p)
} by {
    if p.is_prime and x.coprime(p) {
        if x.congr_mod(Nat.0, p) {
            congr_mod_zero_imp_divides(p, x)
            p.divides(x)
            divides_prime_imp_not_coprime(p, x)
            not x.coprime(p)
            false
        }
    }
}

/// For a prime modulus, nonzero residue classes are exactly the coprime ones.
theorem prime_nonzero_congr_mod_iff_coprime(p: Nat, x: Nat) {
    p.is_prime implies (not x.congr_mod(Nat.0, p) = x.coprime(p))
} by {
    if p.is_prime {
        if not x.congr_mod(Nat.0, p) {
            prime_nonzero_congr_mod_imp_coprime(p, x)
            x.coprime(p)
        }
        if x.coprime(p) {
            prime_coprime_imp_nonzero_congr_mod(p, x)
            not x.congr_mod(Nat.0, p)
        }
        (not x.congr_mod(Nat.0, p) = x.coprime(p)) = true
    }
}

/// A square root that is nonzero modulo a prime gives a unit quadratic residue.
theorem prime_nonzero_root_unit_quadratic_residue(p: Nat, a: Nat, x: Nat) {
    p.is_prime and not x.congr_mod(Nat.0, p) and x.pow(Nat.2).congr_mod(a, p)
        implies is_unit_quadratic_residue_mod(a, p)
} by {
    if p.is_prime and not x.congr_mod(Nat.0, p) and x.pow(Nat.2).congr_mod(a, p) {
        prime_nonzero_congr_mod_imp_coprime(p, x)
        x.coprime(p)
        exists(y: Nat) { y.coprime(p) and y.pow(Nat.2).congr_mod(a, p) }
    }
}

/// A nonzero quadratic residue modulo a prime is a unit quadratic residue.
theorem prime_nonzero_quadratic_residue_is_unit(p: Nat, a: Nat) {
    p.is_prime and is_quadratic_residue_mod(a, p) and not a.congr_mod(Nat.0, p)
        implies is_unit_quadratic_residue_mod(a, p)
} by {
    if p.is_prime and is_quadratic_residue_mod(a, p) and not a.congr_mod(Nat.0, p) {
        let x: Nat satisfy { x.pow(Nat.2).congr_mod(a, p) }
        quadratic_nonzero_target_imp_nonzero_root(a, p, x)
        not x.congr_mod(Nat.0, p)
        prime_nonzero_root_unit_quadratic_residue(p, a, x)
        is_unit_quadratic_residue_mod(a, p)
    }
}

/// A unit quadratic residue modulo a prime is nonzero.
theorem prime_unit_quadratic_residue_nonzero(p: Nat, a: Nat) {
    p.is_prime and is_unit_quadratic_residue_mod(a, p)
        implies not a.congr_mod(Nat.0, p)
} by {
    if p.is_prime and is_unit_quadratic_residue_mod(a, p) {
        unit_quadratic_residue_coprime(a, p)
        a.coprime(p)
        prime_coprime_imp_nonzero_congr_mod(p, a)
        not a.congr_mod(Nat.0, p)
    }
}

/// If `a` is represented by the square of `x`, the order of `x` divides twice
/// the order of `a`.
theorem quadratic_residue_root_order_divides_double_order(a: Nat, n: Nat, x: Nat) {
    n != Nat.0 and x.coprime(n) and x.pow(Nat.2).congr_mod(a, n)
        implies multiplicative_order_mod(x, n).divides(Nat.2 * multiplicative_order_mod(a, n))
} by {
    if n != Nat.0 and x.coprime(n) and x.pow(Nat.2).congr_mod(a, n) {
        square_coprime(x, n)
        x.pow(Nat.2).coprime(n)
        congr_mod_preserves_coprime(x.pow(Nat.2), a, n)
        a.coprime(n)
        multiplicative_order_mod_pow_congr_one(a, n)
        a.pow(multiplicative_order_mod(a, n)).congr_mod(Nat.1, n)
        congr_mod_symm(x.pow(Nat.2), a, n)
        a.congr_mod(x.pow(Nat.2), n)
        congr_mod_pow(a, x.pow(Nat.2), n, multiplicative_order_mod(a, n))
        a.pow(multiplicative_order_mod(a, n)).congr_mod(
            x.pow(Nat.2).pow(multiplicative_order_mod(a, n)), n)
        congr_mod_symm(a.pow(multiplicative_order_mod(a, n)),
            x.pow(Nat.2).pow(multiplicative_order_mod(a, n)), n)
        x.pow(Nat.2).pow(multiplicative_order_mod(a, n)).congr_mod(
            a.pow(multiplicative_order_mod(a, n)), n)
        congr_mod_trans(x.pow(Nat.2).pow(multiplicative_order_mod(a, n)),
            a.pow(multiplicative_order_mod(a, n)), Nat.1, n)
        x.pow(Nat.2).pow(multiplicative_order_mod(a, n)).congr_mod(Nat.1, n)
        exp_mul(x, Nat.2, multiplicative_order_mod(a, n))
        x.pow(Nat.2 * multiplicative_order_mod(a, n)) =
            x.pow(Nat.2).pow(multiplicative_order_mod(a, n))
        x.pow(Nat.2 * multiplicative_order_mod(a, n)).congr_mod(Nat.1, n)
        multiplicative_order_mod_divides_exponent(x, n, Nat.2 * multiplicative_order_mod(a, n))
        multiplicative_order_mod(x, n).divides(Nat.2 * multiplicative_order_mod(a, n))
    }
}

/// A square root of a coprime target is automatically a unit root, so its order
/// divides twice the order of the target.
theorem quadratic_residue_coprime_root_order_divides_double_order(a: Nat, n: Nat, x: Nat) {
    n != Nat.0 and a.coprime(n) and x.pow(Nat.2).congr_mod(a, n)
        implies multiplicative_order_mod(x, n).divides(Nat.2 * multiplicative_order_mod(a, n))
} by {
    if n != Nat.0 and a.coprime(n) and x.pow(Nat.2).congr_mod(a, n) {
        congr_mod_symm(x.pow(Nat.2), a, n)
        a.congr_mod(x.pow(Nat.2), n)
        congr_mod_preserves_coprime(a, x.pow(Nat.2), n)
        x.pow(Nat.2).coprime(n)
        square_coprime_imp_base(x, n)
        x.coprime(n)
        quadratic_residue_root_order_divides_double_order(a, n, x)
        multiplicative_order_mod(x, n).divides(Nat.2 * multiplicative_order_mod(a, n))
    }
}

/// Unit quadratic residues admit an order-constrained square root.
theorem unit_quadratic_residue_order_constraint(a: Nat, n: Nat) {
    n != Nat.0 and is_unit_quadratic_residue_mod(a, n)
        implies exists(x: Nat) {
            x.coprime(n) and x.pow(Nat.2).congr_mod(a, n) and
            multiplicative_order_mod(x, n).divides(Nat.2 * multiplicative_order_mod(a, n))
        }
} by {
    if n != Nat.0 and is_unit_quadratic_residue_mod(a, n) {
        let x: Nat satisfy { x.coprime(n) and x.pow(Nat.2).congr_mod(a, n) }
        quadratic_residue_root_order_divides_double_order(a, n, x)
        exists(y: Nat) {
            y.coprime(n) and y.pow(Nat.2).congr_mod(a, n) and
            multiplicative_order_mod(y, n).divides(Nat.2 * multiplicative_order_mod(a, n))
        }
    }
}

/// Coprime quadratic residues admit an order-constrained square root.
theorem quadratic_residue_coprime_order_constraint(a: Nat, n: Nat) {
    n != Nat.0 and is_quadratic_residue_mod(a, n) and a.coprime(n)
        implies exists(x: Nat) {
            x.coprime(n) and x.pow(Nat.2).congr_mod(a, n) and
            multiplicative_order_mod(x, n).divides(Nat.2 * multiplicative_order_mod(a, n))
        }
} by {
    if n != Nat.0 and is_quadratic_residue_mod(a, n) and a.coprime(n) {
        quadratic_residue_coprime_is_unit(a, n)
        is_unit_quadratic_residue_mod(a, n)
        unit_quadratic_residue_order_constraint(a, n)
        exists(x: Nat) {
            x.coprime(n) and x.pow(Nat.2).congr_mod(a, n) and
            multiplicative_order_mod(x, n).divides(Nat.2 * multiplicative_order_mod(a, n))
        }
    }
}

/// Euler-criterion forward direction in an explicit half-exponent form: if
/// `p = 2*h + 1` and `a` is a unit square modulo `p`, then `a^h ≡ 1 (mod p)`.
theorem euler_criterion_unit_quadratic_residue_forward(p: Nat, h: Nat, a: Nat) {
    p.is_prime and p = Nat.2 * h + Nat.1 and is_unit_quadratic_residue_mod(a, p)
        implies a.pow(h).congr_mod(Nat.1, p)
} by {
    if p.is_prime and p = Nat.2 * h + Nat.1 and is_unit_quadratic_residue_mod(a, p) {
        let x: Nat satisfy { x.coprime(p) and x.pow(Nat.2).congr_mod(a, p) }
        congr_mod_symm(x.pow(Nat.2), a, p)
        a.congr_mod(x.pow(Nat.2), p)
        congr_mod_pow(a, x.pow(Nat.2), p, h)
        a.pow(h).congr_mod(x.pow(Nat.2).pow(h), p)
        exp_mul(x, Nat.2, h)
        x.pow(Nat.2 * h) = x.pow(Nat.2).pow(h)
        x.pow(Nat.2).pow(h).congr_mod(x.pow(Nat.2 * h), p)
        congr_mod_trans(a.pow(h), x.pow(Nat.2).pow(h), x.pow(Nat.2 * h), p)
        a.pow(h).congr_mod(x.pow(Nat.2 * h), p)
        add_imp_sub(Nat.2 * h, Nat.1, p)
        p - Nat.1 = Nat.2 * h
        x.pow(Nat.2 * h) = x.pow(p - Nat.1)
        fermat_euler(p, x)
        x.pow(p - Nat.1).congr_mod(Nat.1, p)
        x.pow(Nat.2 * h).congr_mod(Nat.1, p)
        congr_mod_trans(a.pow(h), x.pow(Nat.2 * h), Nat.1, p)
        a.pow(h).congr_mod(Nat.1, p)
    }
}

/// Euler-criterion forward direction for nonzero quadratic residues modulo an odd prime.
theorem euler_criterion_quadratic_residue_forward(p: Nat, h: Nat, a: Nat) {
    p.is_prime and p = Nat.2 * h + Nat.1 and is_quadratic_residue_mod(a, p)
        and not a.congr_mod(Nat.0, p)
        implies a.pow(h).congr_mod(Nat.1, p)
} by {
    if p.is_prime and p = Nat.2 * h + Nat.1 and is_quadratic_residue_mod(a, p)
        and not a.congr_mod(Nat.0, p) {
        let unit = is_unit_quadratic_residue_mod(a, p)
        prime_nonzero_quadratic_residue_is_unit(p, a)
        unit
        p.is_prime and p = Nat.2 * h + Nat.1 and unit
        euler_criterion_unit_quadratic_residue_forward(p, h, a)
        a.pow(h).congr_mod(Nat.1, p)
    }
}

/// Euler's half-exponent congruence forces the selected order to divide the half-exponent.
theorem euler_condition_imp_order_divides_half(p: Nat, h: Nat, a: Nat) {
    p.is_prime and p = Nat.2 * h + Nat.1 and a.coprime(p)
        and a.pow(h).congr_mod(Nat.1, p)
        implies multiplicative_order_mod(a, p).divides(h)
} by {
    if p.is_prime and p = Nat.2 * h + Nat.1 and a.coprime(p)
        and a.pow(h).congr_mod(Nat.1, p) {
        Nat.1 < p
        p != Nat.0
        multiplicative_order_mod_divides_exponent(a, p, h)
        multiplicative_order_mod(a, p).divides(h)
    }
}

/// If the selected order divides the half-exponent, Euler's half-exponent congruence holds.
theorem order_divides_half_imp_euler_condition(p: Nat, h: Nat, a: Nat) {
    p.is_prime and p = Nat.2 * h + Nat.1 and a.coprime(p)
        and multiplicative_order_mod(a, p).divides(h)
        implies a.pow(h).congr_mod(Nat.1, p)
} by {
    if p.is_prime and p = Nat.2 * h + Nat.1 and a.coprime(p)
        and multiplicative_order_mod(a, p).divides(h) {
        Nat.1 < p
        p != Nat.0
        multiplicative_order_mod_divides_imp_pow_congr_one(a, p, h)
        a.pow(h).congr_mod(Nat.1, p)
    }
}

/// Euler's half-exponent congruence is equivalent to order-divisibility of the
/// half-exponent on the positive-coprime prime-modulus domain.
theorem euler_condition_order_divides_half_iff(p: Nat, h: Nat, a: Nat) {
    p.is_prime and p = Nat.2 * h + Nat.1 and a.coprime(p)
        implies (a.pow(h).congr_mod(Nat.1, p) = multiplicative_order_mod(a, p).divides(h))
} by {
    if p.is_prime and p = Nat.2 * h + Nat.1 and a.coprime(p) {
        if a.pow(h).congr_mod(Nat.1, p) {
            euler_condition_imp_order_divides_half(p, h, a)
            multiplicative_order_mod(a, p).divides(h)
        }
        if multiplicative_order_mod(a, p).divides(h) {
            order_divides_half_imp_euler_condition(p, h, a)
            a.pow(h).congr_mod(Nat.1, p)
        }
        (a.pow(h).congr_mod(Nat.1, p) = multiplicative_order_mod(a, p).divides(h)) = true
    }
}

/// Unit quadratic residues modulo an odd prime have order dividing the half-exponent.
theorem unit_quadratic_residue_order_divides_half(p: Nat, h: Nat, a: Nat) {
    p.is_prime and p = Nat.2 * h + Nat.1 and is_unit_quadratic_residue_mod(a, p)
        implies multiplicative_order_mod(a, p).divides(h)
} by {
    if p.is_prime and p = Nat.2 * h + Nat.1 and is_unit_quadratic_residue_mod(a, p) {
        unit_quadratic_residue_coprime(a, p)
        a.coprime(p)
        euler_criterion_unit_quadratic_residue_forward(p, h, a)
        a.pow(h).congr_mod(Nat.1, p)
        euler_condition_imp_order_divides_half(p, h, a)
        multiplicative_order_mod(a, p).divides(h)
    }
}

/// Nonzero quadratic residues modulo an odd prime have order dividing the half-exponent.
theorem quadratic_residue_order_divides_half(p: Nat, h: Nat, a: Nat) {
    p.is_prime and p = Nat.2 * h + Nat.1 and is_quadratic_residue_mod(a, p)
        and not a.congr_mod(Nat.0, p)
        implies multiplicative_order_mod(a, p).divides(h)
} by {
    if p.is_prime and p = Nat.2 * h + Nat.1 and is_quadratic_residue_mod(a, p)
        and not a.congr_mod(Nat.0, p) {
        prime_nonzero_congr_mod_imp_coprime(p, a)
        a.coprime(p)
        euler_criterion_quadratic_residue_forward(p, h, a)
        a.pow(h).congr_mod(Nat.1, p)
        euler_condition_imp_order_divides_half(p, h, a)
        multiplicative_order_mod(a, p).divides(h)
    }
}

/// For unit quadratic residues modulo an odd prime, Euler's half-exponent
/// congruence is equivalent to the order dividing the half-exponent.
theorem unit_quadratic_residue_euler_condition_order_divides_half_iff(
    p: Nat, h: Nat, a: Nat
) {
    p.is_prime and p = Nat.2 * h + Nat.1 and is_unit_quadratic_residue_mod(a, p)
        implies (a.pow(h).congr_mod(Nat.1, p) =
            multiplicative_order_mod(a, p).divides(h))
} by {
    if p.is_prime and p = Nat.2 * h + Nat.1 and is_unit_quadratic_residue_mod(a, p) {
        unit_quadratic_residue_coprime(a, p)
        a.coprime(p)
        euler_condition_order_divides_half_iff(p, h, a)
        a.pow(h).congr_mod(Nat.1, p) =
            multiplicative_order_mod(a, p).divides(h)
    }
}
