from number_theory.primitive_root import Nat, power_mod_residues,
    full_order_power_mod_residues_unique, powers_cover_units_mod,
    full_multiplicative_order_powers_cover_units_mod
from number_theory.primitive_root_applications import primitive_root_powers_below_order_injective,
    order_two_mod_five, two_coprime_mod_five
from number_theory.primitive_root_discrete_log import is_reduced_power_of_mod,
    reduced_power_index_lt, reduced_power_congr, reduced_power_exists_of_unit_coverage,
    powers_congr_of_common_representative
from number_theory.multiplicative_order import is_multiplicative_order_mod,
    multiplicative_order_mod, multiplicative_order_mod_is_order,
    multiplicative_order_mod_positive, multiplicative_order_pow_congr_one,
    multiplicative_order_mod_divides_exponent
from number_theory.totient import totient_prime, coprime_below_prime
from number_theory.congruence import congr_mod_symm, congr_mod_trans, congr_mod_mul
from number_theory.fermat import fermat_euler
from number_theory.zsigmondy import five_is_prime, divides_suc_pair_imp_one,
    not_divides_of_lt, lt_ne
from number_theory.factorisation import no_proper_divisor_imp_prime
from number_theory.coprime import nat_divides_one_imp_one
from nat import lt_suc, lt_imp_lt_suc, lt_suc_right, lt_not_ref, not_lt_zero,
    trichotomy, lt_imp_lte_suc, lte_and_lt, lte_imp_not_lt,
    divides_sub, divides_self, divides_lte, add_imp_sub, add_imp_sub_left,
    add_comm, add_assoc, add_one_right, add_zero_right, mul_one_left,
    mul_one_right, small_mod, exp_add, exp_one,
    mod_of_decomp, nat_mul_2_2, nat_mul_2_5, nat_mul_2_11, nat_mul_3_3,
    nat_mul_4_2, nat_mul_5_2, nat_mul_4_8, nat_mul_4_4, nat_mul_2_6,
    nat_mul_3_4, nat_mul_4_3, nat_mul_6_2, read_add_read, read_add_single,
    one_plus_one,
    five_plus_four, nat_add_11_2, nat_add_7_4, nat_add_4_2, nat_add_7_2
numerals Nat

// ---------------------------------------------------------------------------
// Applications of primitive roots, part 2.
//
// Throughout, `g` is a primitive root modulo the prime `p`, formalized as
// `p.is_prime and g.coprime(p) and multiplicative_order_mod(g, p) = p - Nat.1`.
// This file proves the fundamental distinctness property of the powers of a
// primitive root, verifies that `2` is a primitive root modulo `5`, `11` and
// `13`, and records the well-definedness of the discrete logarithm.  The
// classical count `φ(p - 1)` of primitive roots modulo `p` and the existence
// theorem for every prime are stated (commented out) at the end, as they need
// counting machinery the library does not yet have.
// ---------------------------------------------------------------------------

/// True when `g` is a primitive root modulo the prime `p`: `g` is coprime to
/// `p` and has multiplicative order `p - 1`, the full order of the group of
/// reduced residues.
define is_primitive_root_mod(g: Nat, p: Nat) -> Bool {
    p.is_prime and g.coprime(p) and multiplicative_order_mod(g, p) = p - Nat.1
}

// ---------------------------------------------------------------------------
// The fundamental property: the powers of a primitive root are distinct.
// ---------------------------------------------------------------------------

/// The powers `g^0, g^1, ..., g^(p-2)` of a primitive root `g` modulo the
/// prime `p` are pairwise distinct modulo `p`: congruent powers with exponents
/// below `p - 1` must have equal exponents.
theorem primitive_root_powers_distinct_mod(p: Nat, g: Nat, i: Nat, j: Nat) {
    p.is_prime and g.coprime(p) and multiplicative_order_mod(g, p) = p - Nat.1
        and i < p - Nat.1 and j < p - Nat.1 and g.pow(i).congr_mod(g.pow(j), p)
        implies i = j
} by {
    if p.is_prime and g.coprime(p) and multiplicative_order_mod(g, p) = p - Nat.1
        and i < p - Nat.1 and j < p - Nat.1 and g.pow(i).congr_mod(g.pow(j), p) {
        primitive_root_powers_below_order_injective(p, g, i, j)
        i = j
    }
}

/// The least residues of the powers `g^0, g^1, ..., g^(p-2)` of a primitive
/// root `g` modulo the prime `p` are all distinct.
theorem primitive_root_power_residues_distinct(p: Nat, g: Nat) {
    p.is_prime and g.coprime(p) and multiplicative_order_mod(g, p) = p - Nat.1
        implies power_mod_residues(g, p).is_unique
} by {
    if p.is_prime and g.coprime(p) and multiplicative_order_mod(g, p) = p - Nat.1 {
        Nat.1 < p
        p != Nat.0
        totient_prime(p)
        p.totient = p - Nat.1
        multiplicative_order_mod(g, p) = p.totient
        full_order_power_mod_residues_unique(g, p)
        power_mod_residues(g, p).is_unique
    }
}

// ---------------------------------------------------------------------------
// Small arithmetic facts used throughout.
// ---------------------------------------------------------------------------

/// `1 + 2 = 3`.
theorem one_plus_two { Nat.1 + Nat.2 = Nat.3 }

/// `1 + 3 = 4`.
theorem one_plus_three { Nat.1 + Nat.3 = Nat.4 }

/// `1 + 4 = 5`.
theorem one_plus_four { Nat.1 + Nat.4 = Nat.5 }

/// `1 + 5 = 6`.
theorem one_plus_five { Nat.1 + Nat.5 = Nat.6 }

/// `1 + 8 = 9`.
theorem one_plus_eight { Nat.1 + Nat.8 = Nat.9 }

/// `1 + 9 = 10`.
theorem one_plus_nine { Nat.1 + Nat.9 = Nat.10 }

/// `1 + 10 = 11`.
theorem one_plus_ten { Nat.1 + Nat.10 = Nat.11 }

/// `2 + 1 = 3`.
theorem two_plus_one { Nat.2 + Nat.1 = Nat.3 }

/// `2 + 2 = 4`.
theorem two_plus_two { Nat.2 + Nat.2 = Nat.4 }

/// `2 + 3 = 5`.
theorem two_plus_three { Nat.2 + Nat.3 = Nat.5 }

/// `2 + 8 = 10`.
theorem two_plus_eight { Nat.2 + Nat.8 = Nat.10 }

/// `2 + 9 = 11`.
theorem two_plus_nine { Nat.2 + Nat.9 = Nat.11 }

/// `3 + 2 = 5`.
theorem three_plus_two { Nat.3 + Nat.2 = Nat.5 }

/// `3 + 3 = 6`.
theorem three_plus_three { Nat.3 + Nat.3 = Nat.6 }

/// `3 + 4 = 7`.
theorem three_plus_four { Nat.3 + Nat.4 = Nat.7 }

/// `3 + 7 = 10`.

/// `3 + 8 = 11`.
theorem three_plus_eight { Nat.3 + Nat.8 = Nat.11 }

/// `3 + 10 = 13`.
theorem three_plus_ten { Nat.3 + Nat.10 = Nat.13 }

/// `4 + 1 = 5`.
theorem four_plus_one { Nat.4 + Nat.1 = Nat.5 }

/// `4 + 6 = 10`.

/// `4 + 7 = 11`.
theorem four_plus_seven { Nat.4 + Nat.7 = Nat.11 }

/// `4 + 9 = 13`.

/// `5 + 1 = 6`.
theorem five_plus_one { Nat.5 + Nat.1 = Nat.6 }

/// `5 + 6 = 11`.
theorem five_plus_six {
    Nat.5 + Nat.6 = Nat.11
} by {
    add_assoc(Nat.5, Nat.5, Nat.1)
    Nat.5 + Nat.5 + Nat.1 = Nat.5 + (Nat.5 + Nat.1)
    one_plus_five
    Nat.1 + Nat.5 = Nat.6
    Nat.5 + Nat.6 = Nat.5 + Nat.5 + Nat.1
    add_assoc(Nat.5, Nat.4, Nat.1)
    Nat.5 + Nat.4 + Nat.1 = Nat.5 + (Nat.4 + Nat.1)
    one_plus_four
    Nat.1 + Nat.4 = Nat.5
    Nat.5 + Nat.5 = Nat.5 + Nat.4 + Nat.1
    five_plus_four
    Nat.5 + Nat.4 = Nat.9
    Nat.5 + Nat.5 = Nat.9 + Nat.1
    add_one_right(Nat.9)
    Nat.9 + Nat.1 = Nat.9.suc
    Nat.9.suc = Nat.10
    Nat.5 + Nat.5 = Nat.10
    Nat.5 + Nat.6 = Nat.10 + Nat.1
    add_one_right(Nat.10)
    Nat.10 + Nat.1 = Nat.10.suc
    Nat.10.suc = Nat.11
    Nat.5 + Nat.6 = Nat.11
}

/// `5 + 8 = 13`.

/// `5 + 9 = 14`.

/// `6 + 7 = 13`.

/// `7 + 5 = 12`.

/// `8 + 4 = 12`.

/// `9 + 3 = 12`.

/// `10 + 1 = 11`.
theorem ten_plus_one { Nat.10 + Nat.1 = Nat.11 }

/// `10 + 2 = 12`.
theorem ten_plus_two { Nat.10 + Nat.2 = Nat.12 }

/// `10 + 4 = 14`.
theorem ten_plus_four { Nat.10 + Nat.4 = Nat.14 }

/// `11 + 1 = 12`.

theorem three_plus_seven {
    Nat.3 + Nat.7 = Nat.10
} by {
    add_comm(Nat.3, Nat.7)
    Nat.3 + Nat.7 = Nat.7 + Nat.3
    add_assoc(Nat.7, Nat.2, Nat.1)
    Nat.7 + Nat.2 + Nat.1 = Nat.7 + (Nat.2 + Nat.1)
    two_plus_one
    Nat.2 + Nat.1 = Nat.3
    Nat.7 + Nat.3 = Nat.7 + Nat.2 + Nat.1
    nat_add_7_2
    Nat.7 + Nat.2 = Nat.9
    Nat.7 + Nat.3 = Nat.9 + Nat.1
    add_one_right(Nat.9)
    Nat.9 + Nat.1 = Nat.9.suc
    Nat.9.suc = Nat.10
    Nat.7 + Nat.3 = Nat.10
    Nat.3 + Nat.7 = Nat.10
}
theorem seven_plus_five {
    Nat.7 + Nat.5 = Nat.12
} by {
    add_assoc(Nat.7, Nat.4, Nat.1)
    Nat.7 + Nat.4 + Nat.1 = Nat.7 + (Nat.4 + Nat.1)
    one_plus_four
    Nat.1 + Nat.4 = Nat.5
    Nat.7 + Nat.5 = Nat.7 + Nat.4 + Nat.1
    nat_add_7_4
    Nat.7 + Nat.4 = Nat.11
    Nat.7 + Nat.5 = Nat.11 + Nat.1
    add_one_right(Nat.11)
    Nat.11 + Nat.1 = Nat.11.suc
    Nat.11.suc = Nat.12
    Nat.7 + Nat.5 = Nat.12
}
theorem eight_plus_four {
    Nat.8 + Nat.4 = Nat.12
} by {
    add_assoc(Nat.8, Nat.3, Nat.1)
    Nat.8 + Nat.3 + Nat.1 = Nat.8 + (Nat.3 + Nat.1)
    one_plus_three
    Nat.1 + Nat.3 = Nat.4
    Nat.8 + Nat.4 = Nat.8 + Nat.3 + Nat.1
    add_comm(Nat.8, Nat.3)
    Nat.8 + Nat.3 = Nat.3 + Nat.8
    three_plus_eight
    Nat.3 + Nat.8 = Nat.11
    Nat.8 + Nat.3 = Nat.11
    Nat.8 + Nat.4 = Nat.11 + Nat.1
    add_one_right(Nat.11)
    Nat.11 + Nat.1 = Nat.11.suc
    Nat.11.suc = Nat.12
    Nat.8 + Nat.4 = Nat.12
}
theorem nine_plus_three {
    Nat.9 + Nat.3 = Nat.12
} by {
    add_assoc(Nat.9, Nat.2, Nat.1)
    Nat.9 + Nat.2 + Nat.1 = Nat.9 + (Nat.2 + Nat.1)
    two_plus_one
    Nat.2 + Nat.1 = Nat.3
    Nat.9 + Nat.3 = Nat.9 + Nat.2 + Nat.1
    add_comm(Nat.9, Nat.2)
    Nat.9 + Nat.2 = Nat.2 + Nat.9
    two_plus_nine
    Nat.2 + Nat.9 = Nat.11
    Nat.9 + Nat.2 = Nat.11
    Nat.9 + Nat.3 = Nat.11 + Nat.1
    add_one_right(Nat.11)
    Nat.11 + Nat.1 = Nat.11.suc
    Nat.11.suc = Nat.12
    Nat.9 + Nat.3 = Nat.12
}
theorem four_plus_six {
    Nat.4 + Nat.6 = Nat.10
} by {
    add_comm(Nat.4, Nat.6)
    Nat.4 + Nat.6 = Nat.6 + Nat.4
    add_assoc(Nat.6, Nat.1, Nat.3)
    Nat.6 + Nat.1 + Nat.3 = Nat.6 + (Nat.1 + Nat.3)
    one_plus_three
    Nat.1 + Nat.3 = Nat.4
    Nat.6 + Nat.4 = Nat.6 + Nat.1 + Nat.3
    add_one_right(Nat.6)
    Nat.6 + Nat.1 = Nat.6.suc
    Nat.6.suc = Nat.7
    Nat.6 + Nat.4 = Nat.7 + Nat.3
    add_comm(Nat.3, Nat.7)
    Nat.3 + Nat.7 = Nat.7 + Nat.3
    three_plus_seven
    Nat.3 + Nat.7 = Nat.10
    Nat.7 + Nat.3 = Nat.10
    Nat.6 + Nat.4 = Nat.10
    Nat.4 + Nat.6 = Nat.10
}
theorem four_plus_nine {
    Nat.4 + Nat.9 = Nat.13
} by {
    add_comm(Nat.4, Nat.9)
    Nat.4 + Nat.9 = Nat.9 + Nat.4
    add_assoc(Nat.9, Nat.3, Nat.1)
    Nat.9 + Nat.3 + Nat.1 = Nat.9 + (Nat.3 + Nat.1)
    one_plus_three
    Nat.1 + Nat.3 = Nat.4
    Nat.9 + Nat.4 = Nat.9 + Nat.3 + Nat.1
    nine_plus_three
    Nat.9 + Nat.3 = Nat.12
    Nat.9 + Nat.4 = Nat.12 + Nat.1
    add_one_right(Nat.12)
    Nat.12 + Nat.1 = Nat.12.suc
    Nat.12.suc = Nat.13
    Nat.9 + Nat.4 = Nat.13
    Nat.4 + Nat.9 = Nat.13
}
theorem five_plus_eight {
    Nat.5 + Nat.8 = Nat.13
} by {
    add_comm(Nat.5, Nat.8)
    Nat.5 + Nat.8 = Nat.8 + Nat.5
    add_assoc(Nat.8, Nat.4, Nat.1)
    Nat.8 + Nat.4 + Nat.1 = Nat.8 + (Nat.4 + Nat.1)
    one_plus_four
    Nat.1 + Nat.4 = Nat.5
    Nat.8 + Nat.5 = Nat.8 + Nat.4 + Nat.1
    eight_plus_four
    Nat.8 + Nat.4 = Nat.12
    Nat.8 + Nat.5 = Nat.12 + Nat.1
    add_one_right(Nat.12)
    Nat.12 + Nat.1 = Nat.12.suc
    Nat.12.suc = Nat.13
    Nat.8 + Nat.5 = Nat.13
    Nat.5 + Nat.8 = Nat.13
}
theorem five_plus_nine {
    Nat.5 + Nat.9 = Nat.14
} by {
    add_comm(Nat.5, Nat.9)
    Nat.5 + Nat.9 = Nat.9 + Nat.5
    add_assoc(Nat.9, Nat.4, Nat.1)
    Nat.9 + Nat.4 + Nat.1 = Nat.9 + (Nat.4 + Nat.1)
    one_plus_four
    Nat.1 + Nat.4 = Nat.5
    Nat.9 + Nat.5 = Nat.9 + Nat.4 + Nat.1
    four_plus_nine
    Nat.4 + Nat.9 = Nat.13
    add_comm(Nat.4, Nat.9)
    Nat.4 + Nat.9 = Nat.9 + Nat.4
    Nat.9 + Nat.4 = Nat.13
    Nat.9 + Nat.5 = Nat.13 + Nat.1
    add_one_right(Nat.13)
    Nat.13 + Nat.1 = Nat.13.suc
    Nat.13.suc = Nat.14
    Nat.9 + Nat.5 = Nat.14
    Nat.5 + Nat.9 = Nat.14
}
theorem six_plus_seven {
    Nat.6 + Nat.7 = Nat.13
} by {
    add_comm(Nat.6, Nat.7)
    Nat.6 + Nat.7 = Nat.7 + Nat.6
    add_assoc(Nat.7, Nat.5, Nat.1)
    Nat.7 + Nat.5 + Nat.1 = Nat.7 + (Nat.5 + Nat.1)
    one_plus_five
    Nat.1 + Nat.5 = Nat.6
    Nat.7 + Nat.6 = Nat.7 + Nat.5 + Nat.1
    seven_plus_five
    Nat.7 + Nat.5 = Nat.12
    Nat.7 + Nat.6 = Nat.12 + Nat.1
    add_one_right(Nat.12)
    Nat.12 + Nat.1 = Nat.12.suc
    Nat.12.suc = Nat.13
    Nat.7 + Nat.6 = Nat.13
    Nat.6 + Nat.7 = Nat.13
}
theorem eleven_plus_one { Nat.11 + Nat.1 = Nat.12 }


/// `1 < 11`.
theorem one_lt_eleven {
    Nat.1 < Nat.11
} by {
    Nat.1 < Nat.2
    lt_imp_lt_suc(Nat.1, Nat.2)
    Nat.1 < Nat.3
    lt_imp_lt_suc(Nat.1, Nat.3)
    Nat.1 < Nat.4
    lt_imp_lt_suc(Nat.1, Nat.4)
    Nat.1 < Nat.5
    lt_imp_lt_suc(Nat.1, Nat.5)
    Nat.1 < Nat.6
    lt_imp_lt_suc(Nat.1, Nat.6)
    Nat.1 < Nat.7
    lt_imp_lt_suc(Nat.1, Nat.7)
    Nat.1 < Nat.8
    lt_imp_lt_suc(Nat.1, Nat.8)
    Nat.1 < Nat.9
    lt_imp_lt_suc(Nat.1, Nat.9)
    Nat.1 < Nat.10
    lt_imp_lt_suc(Nat.1, Nat.10)
    Nat.1 < Nat.11
}

/// `2 < 11`.
theorem two_lt_eleven {
    Nat.2 < Nat.11
} by {
    lt_suc(Nat.2)
    Nat.2 < Nat.3
    lt_imp_lt_suc(Nat.2, Nat.3)
    Nat.2 < Nat.4
    lt_imp_lt_suc(Nat.2, Nat.4)
    Nat.2 < Nat.5
    lt_imp_lt_suc(Nat.2, Nat.5)
    Nat.2 < Nat.6
    lt_imp_lt_suc(Nat.2, Nat.6)
    Nat.2 < Nat.7
    lt_imp_lt_suc(Nat.2, Nat.7)
    Nat.2 < Nat.8
    lt_imp_lt_suc(Nat.2, Nat.8)
    Nat.2 < Nat.9
    lt_imp_lt_suc(Nat.2, Nat.9)
    Nat.2 < Nat.10
    lt_imp_lt_suc(Nat.2, Nat.10)
    Nat.2 < Nat.11
}

/// `4 < 11`.
theorem four_lt_eleven {
    Nat.4 < Nat.11
} by {
    lt_suc(Nat.4)
    Nat.4 < Nat.5
    lt_imp_lt_suc(Nat.4, Nat.5)
    Nat.4 < Nat.6
    lt_imp_lt_suc(Nat.4, Nat.6)
    Nat.4 < Nat.7
    lt_imp_lt_suc(Nat.4, Nat.7)
    Nat.4 < Nat.8
    lt_imp_lt_suc(Nat.4, Nat.8)
    Nat.4 < Nat.9
    lt_imp_lt_suc(Nat.4, Nat.9)
    Nat.4 < Nat.10
    lt_imp_lt_suc(Nat.4, Nat.10)
    Nat.4 < Nat.11
}

/// `10 < 11`.
theorem ten_lt_eleven {
    Nat.10 < Nat.11
} by {
    lt_suc(Nat.10)
    Nat.10 < Nat.11
}

/// `0 < 2`.
theorem lt_zero_two {
    Nat.0 < Nat.2
} by {
    lt_suc(Nat.0)
    Nat.0 < Nat.1
    lt_imp_lt_suc(Nat.0, Nat.1)
    Nat.0 < Nat.2
}

/// `0 < 3`.
theorem lt_zero_three {
    Nat.0 < Nat.3
} by {
    lt_suc(Nat.0)
    Nat.0 < Nat.1
    lt_imp_lt_suc(Nat.0, Nat.1)
    Nat.0 < Nat.2
    lt_imp_lt_suc(Nat.0, Nat.2)
    Nat.0 < Nat.3
}

/// `0 < 4`.
theorem lt_zero_four {
    Nat.0 < Nat.4
} by {
    lt_suc(Nat.0)
    Nat.0 < Nat.1
    lt_imp_lt_suc(Nat.0, Nat.1)
    Nat.0 < Nat.2
    lt_imp_lt_suc(Nat.0, Nat.2)
    Nat.0 < Nat.3
    lt_imp_lt_suc(Nat.0, Nat.3)
    Nat.0 < Nat.4
}

/// `0 < 5`.
theorem lt_zero_five {
    Nat.0 < Nat.5
} by {
    lt_suc(Nat.0)
    Nat.0 < Nat.1
    lt_imp_lt_suc(Nat.0, Nat.1)
    Nat.0 < Nat.2
    lt_imp_lt_suc(Nat.0, Nat.2)
    Nat.0 < Nat.3
    lt_imp_lt_suc(Nat.0, Nat.3)
    Nat.0 < Nat.4
    lt_imp_lt_suc(Nat.0, Nat.4)
    Nat.0 < Nat.5
}

/// `0 < 10`.
theorem lt_zero_ten {
    Nat.0 < Nat.10
} by {
    lt_suc(Nat.0)
    Nat.0 < Nat.1
    lt_imp_lt_suc(Nat.0, Nat.1)
    Nat.0 < Nat.2
    lt_imp_lt_suc(Nat.0, Nat.2)
    Nat.0 < Nat.3
    lt_imp_lt_suc(Nat.0, Nat.3)
    Nat.0 < Nat.4
    lt_imp_lt_suc(Nat.0, Nat.4)
    Nat.0 < Nat.5
    lt_imp_lt_suc(Nat.0, Nat.5)
    Nat.0 < Nat.6
    lt_imp_lt_suc(Nat.0, Nat.6)
    Nat.0 < Nat.7
    lt_imp_lt_suc(Nat.0, Nat.7)
    Nat.0 < Nat.8
    lt_imp_lt_suc(Nat.0, Nat.8)
    Nat.0 < Nat.9
    lt_imp_lt_suc(Nat.0, Nat.9)
    Nat.0 < Nat.10
}

/// `2 < 3`.
theorem lt_two_three {
    Nat.2 < Nat.3
} by {
    lt_suc(Nat.2)
    Nat.2 < Nat.3
}

/// `3 < 4`.
theorem lt_three_four {
    Nat.3 < Nat.4
} by {
    lt_suc(Nat.3)
    Nat.3 < Nat.4
}

/// `5 < 6`.
theorem lt_five_six {
    Nat.5 < Nat.6
} by {
    lt_suc(Nat.5)
    Nat.5 < Nat.6
}

/// `4 < 7`.
theorem lt_four_seven {
    Nat.4 < Nat.7
} by {
    lt_suc(Nat.4)
    Nat.4 < Nat.5
    lt_imp_lt_suc(Nat.4, Nat.5)
    Nat.4 < Nat.6
    lt_imp_lt_suc(Nat.4, Nat.6)
    Nat.4 < Nat.7
}

/// `3 < 8`.
theorem lt_three_eight {
    Nat.3 < Nat.8
} by {
    lt_suc(Nat.3)
    Nat.3 < Nat.4
    lt_imp_lt_suc(Nat.3, Nat.4)
    Nat.3 < Nat.5
    lt_imp_lt_suc(Nat.3, Nat.5)
    Nat.3 < Nat.6
    lt_imp_lt_suc(Nat.3, Nat.6)
    Nat.3 < Nat.7
    lt_imp_lt_suc(Nat.3, Nat.7)
    Nat.3 < Nat.8
}

/// `2 < 9`.
theorem lt_two_nine {
    Nat.2 < Nat.9
} by {
    lt_suc(Nat.2)
    Nat.2 < Nat.3
    lt_imp_lt_suc(Nat.2, Nat.3)
    Nat.2 < Nat.4
    lt_imp_lt_suc(Nat.2, Nat.4)
    Nat.2 < Nat.5
    lt_imp_lt_suc(Nat.2, Nat.5)
    Nat.2 < Nat.6
    lt_imp_lt_suc(Nat.2, Nat.6)
    Nat.2 < Nat.7
    lt_imp_lt_suc(Nat.2, Nat.7)
    Nat.2 < Nat.8
    lt_imp_lt_suc(Nat.2, Nat.8)
    Nat.2 < Nat.9
}

/// `2 < 4`.
theorem lt_two_four {
    Nat.2 < Nat.4
} by {
    lt_suc(Nat.2)
    Nat.2 < Nat.3
    lt_imp_lt_suc(Nat.2, Nat.3)
    Nat.2 < Nat.4
}

/// `4 < 6`.
theorem lt_four_six {
    Nat.4 < Nat.6
} by {
    lt_suc(Nat.4)
    Nat.4 < Nat.5
    lt_imp_lt_suc(Nat.4, Nat.5)
    Nat.4 < Nat.6
}

/// `3 < 7`.
theorem lt_three_seven {
    Nat.3 < Nat.7
} by {
    lt_suc(Nat.3)
    Nat.3 < Nat.4
    lt_imp_lt_suc(Nat.3, Nat.4)
    Nat.3 < Nat.5
    lt_imp_lt_suc(Nat.3, Nat.5)
    Nat.3 < Nat.6
    lt_imp_lt_suc(Nat.3, Nat.6)
    Nat.3 < Nat.7
}

/// `2 < 8`.
theorem lt_two_eight {
    Nat.2 < Nat.8
} by {
    lt_suc(Nat.2)
    Nat.2 < Nat.3
    lt_imp_lt_suc(Nat.2, Nat.3)
    Nat.2 < Nat.4
    lt_imp_lt_suc(Nat.2, Nat.4)
    Nat.2 < Nat.5
    lt_imp_lt_suc(Nat.2, Nat.5)
    Nat.2 < Nat.6
    lt_imp_lt_suc(Nat.2, Nat.6)
    Nat.2 < Nat.7
    lt_imp_lt_suc(Nat.2, Nat.7)
    Nat.2 < Nat.8
}

/// `2 != 1`.
theorem two_ne_one {
    Nat.2 != Nat.1
} by {
    lt_ne(Nat.1, Nat.2, Nat.1)
    one_plus_one
    Nat.1 + Nat.1 = Nat.2
    Nat.1 != Nat.0
    Nat.1 != Nat.2
    Nat.2 != Nat.1
}

/// `3 != 1`.
theorem three_ne_one {
    Nat.3 != Nat.1
} by {
    lt_ne(Nat.1, Nat.3, Nat.2)
    one_plus_two
    Nat.1 + Nat.2 = Nat.3
    Nat.2 != Nat.0
    Nat.1 != Nat.3
    Nat.3 != Nat.1
}

/// `4 != 1`.
theorem four_ne_one {
    Nat.4 != Nat.1
} by {
    lt_ne(Nat.1, Nat.4, Nat.3)
    one_plus_three
    Nat.1 + Nat.3 = Nat.4
    Nat.3 != Nat.0
    Nat.1 != Nat.4
    Nat.4 != Nat.1
}

/// `5 != 1`.
theorem five_ne_one {
    Nat.5 != Nat.1
} by {
    lt_ne(Nat.1, Nat.5, Nat.4)
    one_plus_four
    Nat.1 + Nat.4 = Nat.5
    Nat.4 != Nat.0
    Nat.1 != Nat.5
    Nat.5 != Nat.1
}

/// `6 != 1`.
theorem six_ne_one {
    Nat.6 != Nat.1
} by {
    lt_ne(Nat.1, Nat.6, Nat.5)
    one_plus_five
    Nat.1 + Nat.5 = Nat.6
    Nat.5 != Nat.0
    Nat.1 != Nat.6
    Nat.6 != Nat.1
}

/// `10 != 1`.
theorem ten_ne_one {
    Nat.10 != Nat.1
} by {
    lt_ne(Nat.1, Nat.10, Nat.9)
    one_plus_nine
    Nat.1 + Nat.9 = Nat.10
    Nat.9 != Nat.0
    Nat.1 != Nat.10
    Nat.10 != Nat.1
}

/// `9 != 1`.
theorem nine_ne_one {
    Nat.9 != Nat.1
} by {
    lt_ne(Nat.1, Nat.9, Nat.8)
    one_plus_eight
    Nat.1 + Nat.8 = Nat.9
    Nat.8 != Nat.0
    Nat.1 != Nat.9
    Nat.9 != Nat.1
}

/// `11 - 9 = 2`.
theorem sub_eleven_nine {
    Nat.11 - Nat.9 = Nat.2
} by {
    two_plus_nine
    Nat.2 + Nat.9 = Nat.11
    add_imp_sub(Nat.2, Nat.9, Nat.11)
    Nat.11 - Nat.9 = Nat.2
}

/// `11 - 8 = 3`.
theorem sub_eleven_eight {
    Nat.11 - Nat.8 = Nat.3
} by {
    three_plus_eight
    Nat.3 + Nat.8 = Nat.11
    add_imp_sub(Nat.3, Nat.8, Nat.11)
    Nat.11 - Nat.8 = Nat.3
}

/// `11 - 6 = 5`.
theorem sub_eleven_six {
    Nat.11 - Nat.6 = Nat.5
} by {
    five_plus_six
    Nat.5 + Nat.6 = Nat.11
    add_imp_sub(Nat.5, Nat.6, Nat.11)
    Nat.11 - Nat.6 = Nat.5
}

/// `11 - 7 = 4`.
theorem sub_eleven_seven {
    Nat.11 - Nat.7 = Nat.4
} by {
    four_plus_seven
    Nat.4 + Nat.7 = Nat.11
    add_imp_sub(Nat.4, Nat.7, Nat.11)
    Nat.11 - Nat.7 = Nat.4
}

/// `11 - 10 = 1`.
theorem sub_eleven_ten {
    Nat.11 - Nat.10 = Nat.1
} by {
    one_plus_ten
    Nat.1 + Nat.10 = Nat.11
    add_imp_sub(Nat.1, Nat.10, Nat.11)
    Nat.11 - Nat.10 = Nat.1
}

/// `11 - 1 = 10`.
theorem sub_eleven_one {
    Nat.11 - Nat.1 = Nat.10
} by {
    ten_plus_one
    Nat.10 + Nat.1 = Nat.11
    add_imp_sub(Nat.10, Nat.1, Nat.11)
    Nat.11 - Nat.1 = Nat.10
}

/// `10 - 9 = 1`.
theorem sub_ten_nine {
    Nat.10 - Nat.9 = Nat.1
} by {
    one_plus_nine
    Nat.1 + Nat.9 = Nat.10
    add_imp_sub(Nat.1, Nat.9, Nat.10)
    Nat.10 - Nat.9 = Nat.1
}

/// `10 - 8 = 2`.
theorem sub_ten_eight {
    Nat.10 - Nat.8 = Nat.2
} by {
    two_plus_eight
    Nat.2 + Nat.8 = Nat.10
    add_imp_sub(Nat.2, Nat.8, Nat.10)
    Nat.10 - Nat.8 = Nat.2
}

/// `10 - 6 = 4`.
theorem sub_ten_six {
    Nat.10 - Nat.6 = Nat.4
} by {
    four_plus_six
    Nat.4 + Nat.6 = Nat.10
    add_imp_sub(Nat.4, Nat.6, Nat.10)
    Nat.10 - Nat.6 = Nat.4
}

/// `10 - 7 = 3`.
theorem sub_ten_seven {
    Nat.10 - Nat.7 = Nat.3
} by {
    three_plus_seven
    Nat.3 + Nat.7 = Nat.10
    add_imp_sub(Nat.3, Nat.7, Nat.10)
    Nat.10 - Nat.7 = Nat.3
}

/// `2` divides `10`.
theorem two_divides_ten {
    Nat.2.divides(Nat.10)
} by {
    nat_mul_2_5
    Nat.2 * Nat.5 = Nat.10
    exists(c: Nat) { Nat.2 * c = Nat.10 }
    Nat.2.divides(Nat.10)
}

/// `3` divides `9`.
theorem three_divides_nine {
    Nat.3.divides(Nat.9)
} by {
    nat_mul_3_3
    Nat.3 * Nat.3 = Nat.9
    exists(c: Nat) { Nat.3 * c = Nat.9 }
    Nat.3.divides(Nat.9)
}

/// `4` divides `8`.
theorem four_divides_eight {
    Nat.4.divides(Nat.8)
} by {
    nat_mul_4_2
    Nat.4 * Nat.2 = Nat.8
    exists(c: Nat) { Nat.4 * c = Nat.8 }
    Nat.4.divides(Nat.8)
}

/// `5` divides `10`.
theorem five_divides_ten {
    Nat.5.divides(Nat.10)
} by {
    nat_mul_5_2
    Nat.5 * Nat.2 = Nat.10
    exists(c: Nat) { Nat.5 * c = Nat.10 }
    Nat.5.divides(Nat.10)
}

/// Two does not divide eleven.
theorem not_two_divides_eleven {
    not Nat.2.divides(Nat.11)
} by {
    if Nat.2.divides(Nat.11) {
        two_divides_ten
        Nat.2.divides(Nat.10)
        divides_suc_pair_imp_one(Nat.2, Nat.10)
        Nat.2 = Nat.1
        two_ne_one
        false
    }
}

/// Three does not divide eleven.
theorem not_three_divides_eleven {
    not Nat.3.divides(Nat.11)
} by {
    if Nat.3.divides(Nat.11) {
        three_divides_nine
        Nat.3.divides(Nat.9)
        divides_sub(Nat.11, Nat.9, Nat.3)
        Nat.3.divides(Nat.11 - Nat.9)
        sub_eleven_nine
        Nat.11 - Nat.9 = Nat.2
        Nat.3.divides(Nat.2)
        lt_zero_two
        Nat.0 < Nat.2
        lt_two_three
        Nat.2 < Nat.3
        not_divides_of_lt(Nat.3, Nat.2)
        not Nat.3.divides(Nat.2)
        false
    }
}

/// Four does not divide eleven.
theorem not_four_divides_eleven {
    not Nat.4.divides(Nat.11)
} by {
    if Nat.4.divides(Nat.11) {
        four_divides_eight
        Nat.4.divides(Nat.8)
        divides_sub(Nat.11, Nat.8, Nat.4)
        Nat.4.divides(Nat.11 - Nat.8)
        sub_eleven_eight
        Nat.11 - Nat.8 = Nat.3
        Nat.4.divides(Nat.3)
        lt_zero_three
        Nat.0 < Nat.3
        lt_three_four
        Nat.3 < Nat.4
        not_divides_of_lt(Nat.4, Nat.3)
        not Nat.4.divides(Nat.3)
        false
    }
}

/// Five does not divide eleven.
theorem not_five_divides_eleven {
    not Nat.5.divides(Nat.11)
} by {
    if Nat.5.divides(Nat.11) {
        five_divides_ten
        Nat.5.divides(Nat.10)
        divides_suc_pair_imp_one(Nat.5, Nat.10)
        Nat.5 = Nat.1
        five_ne_one
        false
    }
}

/// Six does not divide eleven.
theorem not_six_divides_eleven {
    not Nat.6.divides(Nat.11)
} by {
    if Nat.6.divides(Nat.11) {
        divides_self(Nat.6)
        Nat.6.divides(Nat.6)
        divides_sub(Nat.11, Nat.6, Nat.6)
        Nat.6.divides(Nat.11 - Nat.6)
        sub_eleven_six
        Nat.11 - Nat.6 = Nat.5
        Nat.6.divides(Nat.5)
        lt_zero_five
        Nat.0 < Nat.5
        lt_five_six
        Nat.5 < Nat.6
        not_divides_of_lt(Nat.6, Nat.5)
        not Nat.6.divides(Nat.5)
        false
    }
}

/// Seven does not divide eleven.
theorem not_seven_divides_eleven {
    not Nat.7.divides(Nat.11)
} by {
    if Nat.7.divides(Nat.11) {
        divides_self(Nat.7)
        Nat.7.divides(Nat.7)
        divides_sub(Nat.11, Nat.7, Nat.7)
        Nat.7.divides(Nat.11 - Nat.7)
        sub_eleven_seven
        Nat.11 - Nat.7 = Nat.4
        Nat.7.divides(Nat.4)
        lt_zero_four
        Nat.0 < Nat.4
        lt_four_seven
        Nat.4 < Nat.7
        not_divides_of_lt(Nat.7, Nat.4)
        not Nat.7.divides(Nat.4)
        false
    }
}

/// Eight does not divide eleven.
theorem not_eight_divides_eleven {
    not Nat.8.divides(Nat.11)
} by {
    if Nat.8.divides(Nat.11) {
        divides_self(Nat.8)
        Nat.8.divides(Nat.8)
        divides_sub(Nat.11, Nat.8, Nat.8)
        Nat.8.divides(Nat.11 - Nat.8)
        sub_eleven_eight
        Nat.11 - Nat.8 = Nat.3
        Nat.8.divides(Nat.3)
        lt_zero_three
        Nat.0 < Nat.3
        lt_three_eight
        Nat.3 < Nat.8
        not_divides_of_lt(Nat.8, Nat.3)
        not Nat.8.divides(Nat.3)
        false
    }
}

/// Nine does not divide eleven.
theorem not_nine_divides_eleven {
    not Nat.9.divides(Nat.11)
} by {
    if Nat.9.divides(Nat.11) {
        divides_self(Nat.9)
        Nat.9.divides(Nat.9)
        divides_sub(Nat.11, Nat.9, Nat.9)
        Nat.9.divides(Nat.11 - Nat.9)
        sub_eleven_nine
        Nat.11 - Nat.9 = Nat.2
        Nat.9.divides(Nat.2)
        lt_zero_two
        Nat.0 < Nat.2
        lt_two_nine
        Nat.2 < Nat.9
        not_divides_of_lt(Nat.9, Nat.2)
        not Nat.9.divides(Nat.2)
        false
    }
}

/// Ten does not divide eleven.
theorem not_ten_divides_eleven {
    not Nat.10.divides(Nat.11)
} by {
    if Nat.10.divides(Nat.11) {
        mul_one_right(Nat.10)
        Nat.10 * Nat.1 = Nat.10
        exists(c: Nat) { Nat.10 * c = Nat.10 }
        Nat.10.divides(Nat.10)
        divides_sub(Nat.11, Nat.10, Nat.10)
        Nat.10.divides(Nat.11 - Nat.10)
        sub_eleven_ten
        Nat.11 - Nat.10 = Nat.1
        Nat.10.divides(Nat.1)
        nat_divides_one_imp_one(Nat.10)
        Nat.10 = Nat.1
        ten_ne_one
        false
    }
}

/// `1 < k` and `k < 3` forces `k = 2`.
theorem k_range_min(k: Nat) {
    Nat.1 < k and k < Nat.3 implies k = Nat.2
} by {
    if Nat.1 < k and k < Nat.3 {
        lt_suc_right(k, Nat.2)
        k = Nat.2 or k < Nat.2
        if k < Nat.2 {
            lt_imp_lte_suc(Nat.1, k)
            Nat.2 <= k
            lte_and_lt(Nat.2, k, Nat.2)
            Nat.2 < Nat.2
            lt_not_ref(Nat.2)
            false
        } else {
            k = Nat.2
            k = Nat.2
        }
    }
}

/// `1 < k` and `k < 4` forces `k` into `{2, ..., 3}`.
theorem k_range_two_3(k: Nat) {
    Nat.1 < k and k < Nat.4 implies (k = Nat.2 or k = Nat.3)
} by {
    if Nat.1 < k and k < Nat.4 {
        lt_suc_right(k, Nat.3)
        k = Nat.3 or k < Nat.3
        if k < Nat.3 {
            k_range_min(k)
            k = Nat.2
            k = Nat.2 or k = Nat.3
        } else {
            k = Nat.3
            k = Nat.2 or k = Nat.3
        }
    }
}

/// `1 < k` and `k < 5` forces `k` into `{2, ..., 4}`.
theorem k_range_two_4(k: Nat) {
    Nat.1 < k and k < Nat.5 implies (k = Nat.2 or k = Nat.3 or k = Nat.4)
} by {
    if Nat.1 < k and k < Nat.5 {
        lt_suc_right(k, Nat.4)
        k = Nat.4 or k < Nat.4
        if k < Nat.4 {
            k_range_two_3(k)
            k = Nat.2 or k = Nat.3
            k = Nat.2 or k = Nat.3 or k = Nat.4
        } else {
            k = Nat.4
            k = Nat.2 or k = Nat.3 or k = Nat.4
        }
    }
}

/// `1 < k` and `k < 6` forces `k` into `{2, ..., 5}`.
theorem k_range_two_5(k: Nat) {
    Nat.1 < k and k < Nat.6 implies (k = Nat.2 or k = Nat.3 or k = Nat.4 or k = Nat.5)
} by {
    if Nat.1 < k and k < Nat.6 {
        lt_suc_right(k, Nat.5)
        k = Nat.5 or k < Nat.5
        if k < Nat.5 {
            k_range_two_4(k)
            k = Nat.2 or k = Nat.3 or k = Nat.4
            k = Nat.2 or k = Nat.3 or k = Nat.4 or k = Nat.5
        } else {
            k = Nat.5
            k = Nat.2 or k = Nat.3 or k = Nat.4 or k = Nat.5
        }
    }
}

/// `1 < k` and `k < 7` forces `k` into `{2, ..., 6}`.
theorem k_range_two_6(k: Nat) {
    Nat.1 < k and k < Nat.7 implies (k = Nat.2 or k = Nat.3 or k = Nat.4 or k = Nat.5 or k = Nat.6)
} by {
    if Nat.1 < k and k < Nat.7 {
        lt_suc_right(k, Nat.6)
        k = Nat.6 or k < Nat.6
        if k < Nat.6 {
            k_range_two_5(k)
            k = Nat.2 or k = Nat.3 or k = Nat.4 or k = Nat.5
            k = Nat.2 or k = Nat.3 or k = Nat.4 or k = Nat.5 or k = Nat.6
        } else {
            k = Nat.6
            k = Nat.2 or k = Nat.3 or k = Nat.4 or k = Nat.5 or k = Nat.6
        }
    }
}

/// `1 < k` and `k < 8` forces `k` into `{2, ..., 7}`.
theorem k_range_two_7(k: Nat) {
    Nat.1 < k and k < Nat.8 implies (k = Nat.2 or k = Nat.3 or k = Nat.4 or k = Nat.5 or k = Nat.6 or k = Nat.7)
} by {
    if Nat.1 < k and k < Nat.8 {
        lt_suc_right(k, Nat.7)
        k = Nat.7 or k < Nat.7
        if k < Nat.7 {
            k_range_two_6(k)
            k = Nat.2 or k = Nat.3 or k = Nat.4 or k = Nat.5 or k = Nat.6
            k = Nat.2 or k = Nat.3 or k = Nat.4 or k = Nat.5 or k = Nat.6 or k = Nat.7
        } else {
            k = Nat.7
            k = Nat.2 or k = Nat.3 or k = Nat.4 or k = Nat.5 or k = Nat.6 or k = Nat.7
        }
    }
}

/// `1 < k` and `k < 9` forces `k` into `{2, ..., 8}`.
theorem k_range_two_8(k: Nat) {
    Nat.1 < k and k < Nat.9 implies (k = Nat.2 or k = Nat.3 or k = Nat.4 or k = Nat.5 or k = Nat.6 or k = Nat.7 or k = Nat.8)
} by {
    if Nat.1 < k and k < Nat.9 {
        lt_suc_right(k, Nat.8)
        k = Nat.8 or k < Nat.8
        if k < Nat.8 {
            k_range_two_7(k)
            k = Nat.2 or k = Nat.3 or k = Nat.4 or k = Nat.5 or k = Nat.6 or k = Nat.7
            k = Nat.2 or k = Nat.3 or k = Nat.4 or k = Nat.5 or k = Nat.6 or k = Nat.7 or k = Nat.8
        } else {
            k = Nat.8
            k = Nat.2 or k = Nat.3 or k = Nat.4 or k = Nat.5 or k = Nat.6 or k = Nat.7 or k = Nat.8
        }
    }
}

/// `1 < k` and `k < 10` forces `k` into `{2, ..., 9}`.
theorem k_range_two_9(k: Nat) {
    Nat.1 < k and k < Nat.10 implies (k = Nat.2 or k = Nat.3 or k = Nat.4 or k = Nat.5 or k = Nat.6 or k = Nat.7 or k = Nat.8 or k = Nat.9)
} by {
    if Nat.1 < k and k < Nat.10 {
        lt_suc_right(k, Nat.9)
        k = Nat.9 or k < Nat.9
        if k < Nat.9 {
            k_range_two_8(k)
            k = Nat.2 or k = Nat.3 or k = Nat.4 or k = Nat.5 or k = Nat.6 or k = Nat.7 or k = Nat.8
            k = Nat.2 or k = Nat.3 or k = Nat.4 or k = Nat.5 or k = Nat.6 or k = Nat.7 or k = Nat.8 or k = Nat.9
        } else {
            k = Nat.9
            k = Nat.2 or k = Nat.3 or k = Nat.4 or k = Nat.5 or k = Nat.6 or k = Nat.7 or k = Nat.8 or k = Nat.9
        }
    }
}

/// `1 < k` and `k < 11` forces `k` into `{2, ..., 10}`.
theorem k_range_two_ten(k: Nat) {
    Nat.1 < k and k < Nat.11 implies (k = Nat.2 or k = Nat.3 or k = Nat.4 or k = Nat.5 or k = Nat.6 or k = Nat.7 or k = Nat.8 or k = Nat.9 or k = Nat.10)
} by {
    if Nat.1 < k and k < Nat.11 {
        lt_suc_right(k, Nat.10)
        k = Nat.10 or k < Nat.10
        if k < Nat.10 {
            k_range_two_9(k)
            k = Nat.2 or k = Nat.3 or k = Nat.4 or k = Nat.5 or k = Nat.6 or k = Nat.7 or k = Nat.8 or k = Nat.9
            k = Nat.2 or k = Nat.3 or k = Nat.4 or k = Nat.5 or k = Nat.6 or k = Nat.7 or k = Nat.8 or k = Nat.9 or k = Nat.10
        } else {
            k = Nat.10
            k = Nat.2 or k = Nat.3 or k = Nat.4 or k = Nat.5 or k = Nat.6 or k = Nat.7 or k = Nat.8 or k = Nat.9 or k = Nat.10
        }
    }
}

/// No number strictly between 1 and 11 divides 11 (by cases).
theorem eleven_not_divides_by_case(k: Nat) {
    Nat.1 < k and k < Nat.11 and (k = Nat.2 or k = Nat.3 or k = Nat.4 or k = Nat.5 or
        k = Nat.6 or k = Nat.7 or k = Nat.8 or k = Nat.9 or k = Nat.10)
        implies not k.divides(Nat.11)
} by {
    if Nat.1 < k and k < Nat.11 and (k = Nat.2 or k = Nat.3 or k = Nat.4 or k = Nat.5 or
        k = Nat.6 or k = Nat.7 or k = Nat.8 or k = Nat.9 or k = Nat.10) {
        if k = Nat.2 {
            not_two_divides_eleven
            not Nat.2.divides(Nat.11)
            not k.divides(Nat.11)
        } else {
            (k = Nat.3 or k = Nat.4 or k = Nat.5 or k = Nat.6 or k = Nat.7 or k = Nat.8 or k = Nat.9 or k = Nat.10)
        if k = Nat.3 {
            not_three_divides_eleven
            not Nat.3.divides(Nat.11)
            not k.divides(Nat.11)
        } else {
            (k = Nat.4 or k = Nat.5 or k = Nat.6 or k = Nat.7 or k = Nat.8 or k = Nat.9 or k = Nat.10)
        if k = Nat.4 {
            not_four_divides_eleven
            not Nat.4.divides(Nat.11)
            not k.divides(Nat.11)
        } else {
            (k = Nat.5 or k = Nat.6 or k = Nat.7 or k = Nat.8 or k = Nat.9 or k = Nat.10)
        if k = Nat.5 {
            not_five_divides_eleven
            not Nat.5.divides(Nat.11)
            not k.divides(Nat.11)
        } else {
            (k = Nat.6 or k = Nat.7 or k = Nat.8 or k = Nat.9 or k = Nat.10)
        if k = Nat.6 {
            not_six_divides_eleven
            not Nat.6.divides(Nat.11)
            not k.divides(Nat.11)
        } else {
            (k = Nat.7 or k = Nat.8 or k = Nat.9 or k = Nat.10)
        if k = Nat.7 {
            not_seven_divides_eleven
            not Nat.7.divides(Nat.11)
            not k.divides(Nat.11)
        } else {
            (k = Nat.8 or k = Nat.9 or k = Nat.10)
        if k = Nat.8 {
            not_eight_divides_eleven
            not Nat.8.divides(Nat.11)
            not k.divides(Nat.11)
        } else {
            (k = Nat.9 or k = Nat.10)
        if k = Nat.9 {
            not_nine_divides_eleven
            not Nat.9.divides(Nat.11)
            not k.divides(Nat.11)
        } else {
            k = Nat.10
            not_ten_divides_eleven
            not Nat.10.divides(Nat.11)
            not k.divides(Nat.11)
        }
        }
        }
        }
        }
        }
        }
        }
    }
}

/// No number strictly between 1 and 11 divides 11.
theorem eleven_no_proper_divisor(k: Nat) {
    Nat.1 < k and k < Nat.11 implies not k.divides(Nat.11)
} by {
    if Nat.1 < k and k < Nat.11 {
        k_range_two_ten(k)
        k = Nat.2 or k = Nat.3 or k = Nat.4 or k = Nat.5 or
            k = Nat.6 or k = Nat.7 or k = Nat.8 or k = Nat.9 or k = Nat.10
        eleven_not_divides_by_case(k)
        not k.divides(Nat.11)
    }
}


/// Eleven is prime.
theorem eleven_is_prime {
    Nat.11.is_prime
} by {
    one_lt_eleven
    Nat.1 < Nat.11
    forall(k: Nat) {
        eleven_no_proper_divisor(k)
    }
    no_proper_divisor_imp_prime(Nat.11)
}

/// `2` is coprime to `11`.
theorem two_coprime_mod_eleven {
    Nat.2.coprime(Nat.11)
} by {
    eleven_is_prime
    Nat.11.is_prime
    Nat.1 <= Nat.2
    two_lt_eleven
    Nat.2 < Nat.11
    coprime_below_prime(Nat.11, Nat.2)
    Nat.2.coprime(Nat.11)
}

/// `1 mod 11 = 1`.
theorem mod_one_mod_eleven {
    Nat.1.mod(Nat.11) = Nat.1
} by {
    one_lt_eleven
    Nat.1 < Nat.11
    small_mod(Nat.1, Nat.11)
}

/// `2 mod 11 = 2`.
theorem mod_two_mod_eleven {
    Nat.2.mod(Nat.11) = Nat.2
} by {
    two_lt_eleven
    Nat.2 < Nat.11
    small_mod(Nat.2, Nat.11)
}

/// `4 mod 11 = 4`.
theorem mod_four_mod_eleven {
    Nat.4.mod(Nat.11) = Nat.4
} by {
    four_lt_eleven
    Nat.4 < Nat.11
    small_mod(Nat.4, Nat.11)
}

/// `2^1 = 2`.
theorem pow_two_one {
    Nat.2.pow(Nat.1) = Nat.2
} by {
    exp_one(Nat.2)
}

/// `2^2 = 4`.
theorem pow_two_two {
    Nat.2.pow(Nat.2) = Nat.4
} by {
    exp_add(Nat.2, Nat.1, Nat.1)
    Nat.2.pow(Nat.1 + Nat.1) = Nat.2.pow(Nat.1) * Nat.2.pow(Nat.1)
    one_plus_one
    Nat.1 + Nat.1 = Nat.2
    Nat.2.pow(Nat.2) = Nat.2.pow(Nat.1) * Nat.2.pow(Nat.1)
    pow_two_one
    Nat.2.pow(Nat.1) = Nat.2
    Nat.2.pow(Nat.2) = Nat.2 * Nat.2
    nat_mul_2_2
    Nat.2 * Nat.2 = Nat.4
    Nat.2.pow(Nat.2) = Nat.4
}

/// `2^3 = 8`.
theorem pow_two_three {
    Nat.2.pow(Nat.3) = Nat.8
} by {
    exp_add(Nat.2, Nat.2, Nat.1)
    Nat.2.pow(Nat.2 + Nat.1) = Nat.2.pow(Nat.2) * Nat.2.pow(Nat.1)
    two_plus_one
    Nat.2 + Nat.1 = Nat.3
    Nat.2.pow(Nat.3) = Nat.2.pow(Nat.2) * Nat.2.pow(Nat.1)
    pow_two_two
    Nat.2.pow(Nat.2) = Nat.4
    pow_two_one
    Nat.2.pow(Nat.1) = Nat.2
    Nat.2.pow(Nat.3) = Nat.4 * Nat.2
    nat_mul_4_2
    Nat.4 * Nat.2 = Nat.8
    Nat.2.pow(Nat.3) = Nat.8
}

/// `2^5 = 32`.
theorem pow_two_five {
    Nat.2.pow(Nat.5) = Nat.32
} by {
    exp_add(Nat.2, Nat.2, Nat.3)
    Nat.2.pow(Nat.2 + Nat.3) = Nat.2.pow(Nat.2) * Nat.2.pow(Nat.3)
    two_plus_three
    Nat.2 + Nat.3 = Nat.5
    Nat.2.pow(Nat.5) = Nat.2.pow(Nat.2) * Nat.2.pow(Nat.3)
    pow_two_two
    Nat.2.pow(Nat.2) = Nat.4
    pow_two_three
    Nat.2.pow(Nat.3) = Nat.8
    Nat.2.pow(Nat.5) = Nat.4 * Nat.8
    nat_mul_4_8
    Nat.4 * Nat.8 = Nat.32
    Nat.2.pow(Nat.5) = Nat.32
}

/// `22 + 10 = 32`.
theorem nat_add_22_10 {
    Nat.22 + Nat.10 = Nat.32
} by {
    Nat.22 = Nat.2.read(Nat.2)
    Nat.10 = Nat.1.read(Nat.0)
    read_add_read(Nat.2, Nat.2, Nat.1, Nat.0)
    Nat.2.read(Nat.2) + Nat.1.read(Nat.0) = (Nat.2 + Nat.1).read(Nat.2 + Nat.0)
    two_plus_one
    Nat.2 + Nat.1 = Nat.3
    add_zero_right(Nat.2)
    Nat.2 + Nat.0 = Nat.2
    Nat.3.read(Nat.2) = Nat.32
}

/// `32 ≡ 10 (mod 11)`.
theorem congr_thirty_two_mod_eleven {
    Nat.32.congr_mod(Nat.10, Nat.11)
} by {
    nat_mul_2_11
    Nat.2 * Nat.11 = Nat.22
    nat_add_22_10
    Nat.22 + Nat.10 = Nat.32
    Nat.2 * Nat.11 + Nat.10 = Nat.32
    ten_lt_eleven
    Nat.10 < Nat.11
    mod_of_decomp(Nat.2, Nat.10, Nat.11)
    (Nat.2 * Nat.11 + Nat.10).mod(Nat.11) = Nat.10
    Nat.32.mod(Nat.11) = Nat.10
    small_mod(Nat.10, Nat.11)
    Nat.10.mod(Nat.11) = Nat.10
    Nat.32.mod(Nat.11) = Nat.10.mod(Nat.11)
    Nat.32.congr_mod(Nat.10, Nat.11)
}

/// `2^5 ≡ 10 (mod 11)`.
theorem congr_two_pow_five_mod_eleven {
    Nat.2.pow(Nat.5).congr_mod(Nat.10, Nat.11)
} by {
    pow_two_five
    Nat.2.pow(Nat.5) = Nat.32
    congr_thirty_two_mod_eleven
    Nat.32.congr_mod(Nat.10, Nat.11)
    Nat.2.pow(Nat.5).congr_mod(Nat.10, Nat.11)
}

/// `10 ≢ 1 (mod 11)`.
theorem not_congr_ten_mod_eleven {
    not Nat.10.congr_mod(Nat.1, Nat.11)
} by {
    ten_lt_eleven
    Nat.10 < Nat.11
    small_mod(Nat.10, Nat.11)
    Nat.10.mod(Nat.11) = Nat.10
    one_lt_eleven
    Nat.1 < Nat.11
    small_mod(Nat.1, Nat.11)
    Nat.1.mod(Nat.11) = Nat.1
    if Nat.10.congr_mod(Nat.1, Nat.11) {
        Nat.10.mod(Nat.11) = Nat.1.mod(Nat.11)
        Nat.10 = Nat.1
        ten_ne_one
        false
    }
}

/// `2^1 ≢ 1 (mod 11)`.
theorem not_congr_two_pow_one_mod_eleven {
    not Nat.2.pow(Nat.1).congr_mod(Nat.1, Nat.11)
} by {
    mod_two_mod_eleven
    Nat.2.mod(Nat.11) = Nat.2
    mod_one_mod_eleven
    Nat.1.mod(Nat.11) = Nat.1
    if Nat.2.pow(Nat.1).congr_mod(Nat.1, Nat.11) {
        pow_two_one
        Nat.2.pow(Nat.1) = Nat.2
        Nat.2.congr_mod(Nat.1, Nat.11)
        Nat.2.mod(Nat.11) = Nat.1.mod(Nat.11)
        Nat.2 = Nat.1
        two_ne_one
        false
    }
}

/// `2^2 ≢ 1 (mod 11)`.
theorem not_congr_two_pow_two_mod_eleven {
    not Nat.2.pow(Nat.2).congr_mod(Nat.1, Nat.11)
} by {
    mod_four_mod_eleven
    Nat.4.mod(Nat.11) = Nat.4
    mod_one_mod_eleven
    Nat.1.mod(Nat.11) = Nat.1
    if Nat.2.pow(Nat.2).congr_mod(Nat.1, Nat.11) {
        pow_two_two
        Nat.2.pow(Nat.2) = Nat.4
        Nat.4.congr_mod(Nat.1, Nat.11)
        Nat.4.mod(Nat.11) = Nat.1.mod(Nat.11)
        Nat.4 = Nat.1
        four_ne_one
        false
    }
}

/// `2^5 ≢ 1 (mod 11)`.
theorem not_congr_two_pow_five_mod_eleven {
    not Nat.2.pow(Nat.5).congr_mod(Nat.1, Nat.11)
} by {
    congr_two_pow_five_mod_eleven
    Nat.2.pow(Nat.5).congr_mod(Nat.10, Nat.11)
    if Nat.2.pow(Nat.5).congr_mod(Nat.1, Nat.11) {
        congr_mod_symm(Nat.2.pow(Nat.5), Nat.10, Nat.11)
        Nat.10.congr_mod(Nat.2.pow(Nat.5), Nat.11)
        congr_mod_trans(Nat.10, Nat.2.pow(Nat.5), Nat.1, Nat.11)
        Nat.10.congr_mod(Nat.1, Nat.11)
        not_congr_ten_mod_eleven
        false
    }
}

/// `3` does not divide `10`.
theorem not_three_divides_ten {
    not Nat.3.divides(Nat.10)
} by {
    if Nat.3.divides(Nat.10) {
        three_divides_nine
        Nat.3.divides(Nat.9)
        divides_sub(Nat.10, Nat.9, Nat.3)
        Nat.3.divides(Nat.10 - Nat.9)
        sub_ten_nine
        Nat.10 - Nat.9 = Nat.1
        Nat.3.divides(Nat.1)
        nat_divides_one_imp_one(Nat.3)
        Nat.3 = Nat.1
        three_ne_one
        false
    }
}

/// `4` does not divide `10`.
theorem not_four_divides_ten {
    not Nat.4.divides(Nat.10)
} by {
    if Nat.4.divides(Nat.10) {
        four_divides_eight
        Nat.4.divides(Nat.8)
        divides_sub(Nat.10, Nat.8, Nat.4)
        Nat.4.divides(Nat.10 - Nat.8)
        sub_ten_eight
        Nat.10 - Nat.8 = Nat.2
        Nat.4.divides(Nat.2)
        lt_zero_two
        Nat.0 < Nat.2
        lt_two_four
        Nat.2 < Nat.4
        not_divides_of_lt(Nat.4, Nat.2)
        not Nat.4.divides(Nat.2)
        false
    }
}

/// `6` does not divide `10`.
theorem not_six_divides_ten {
    not Nat.6.divides(Nat.10)
} by {
    if Nat.6.divides(Nat.10) {
        divides_self(Nat.6)
        Nat.6.divides(Nat.6)
        divides_sub(Nat.10, Nat.6, Nat.6)
        Nat.6.divides(Nat.10 - Nat.6)
        sub_ten_six
        Nat.10 - Nat.6 = Nat.4
        Nat.6.divides(Nat.4)
        lt_zero_four
        Nat.0 < Nat.4
        lt_four_six
        Nat.4 < Nat.6
        not_divides_of_lt(Nat.6, Nat.4)
        not Nat.6.divides(Nat.4)
        false
    }
}

/// `7` does not divide `10`.
theorem not_seven_divides_ten {
    not Nat.7.divides(Nat.10)
} by {
    if Nat.7.divides(Nat.10) {
        divides_self(Nat.7)
        Nat.7.divides(Nat.7)
        divides_sub(Nat.10, Nat.7, Nat.7)
        Nat.7.divides(Nat.10 - Nat.7)
        sub_ten_seven
        Nat.10 - Nat.7 = Nat.3
        Nat.7.divides(Nat.3)
        lt_zero_three
        Nat.0 < Nat.3
        lt_three_seven
        Nat.3 < Nat.7
        not_divides_of_lt(Nat.7, Nat.3)
        not Nat.7.divides(Nat.3)
        false
    }
}

/// `8` does not divide `10`.
theorem not_eight_divides_ten {
    not Nat.8.divides(Nat.10)
} by {
    if Nat.8.divides(Nat.10) {
        divides_self(Nat.8)
        Nat.8.divides(Nat.8)
        divides_sub(Nat.10, Nat.8, Nat.8)
        Nat.8.divides(Nat.10 - Nat.8)
        sub_ten_eight
        Nat.10 - Nat.8 = Nat.2
        Nat.8.divides(Nat.2)
        lt_zero_two
        Nat.0 < Nat.2
        lt_two_eight
        Nat.2 < Nat.8
        not_divides_of_lt(Nat.8, Nat.2)
        not Nat.8.divides(Nat.2)
        false
    }
}

/// `9` does not divide `10`.
theorem not_nine_divides_ten {
    not Nat.9.divides(Nat.10)
} by {
    if Nat.9.divides(Nat.10) {
        divides_self(Nat.9)
        Nat.9.divides(Nat.9)
        divides_sub(Nat.10, Nat.9, Nat.9)
        Nat.9.divides(Nat.10 - Nat.9)
        sub_ten_nine
        Nat.10 - Nat.9 = Nat.1
        Nat.9.divides(Nat.1)
        nat_divides_one_imp_one(Nat.9)
        Nat.9 = Nat.1
        nine_ne_one
        false
    }
}

/// `2` has multiplicative order `10` modulo `11`.
theorem order_two_mod_eleven {
    multiplicative_order_mod(Nat.2, Nat.11) = Nat.10
} by {
    Nat.11 != Nat.0
    two_coprime_mod_eleven
    Nat.2.coprime(Nat.11)
    fermat_euler(Nat.11, Nat.2)
    Nat.2.pow(Nat.11 - Nat.1).congr_mod(Nat.1, Nat.11)
    sub_eleven_one
    Nat.11 - Nat.1 = Nat.10
    Nat.2.pow(Nat.10).congr_mod(Nat.1, Nat.11)
    multiplicative_order_mod_is_order(Nat.2, Nat.11)
    is_multiplicative_order_mod(Nat.2, Nat.11, multiplicative_order_mod(Nat.2, Nat.11))
    multiplicative_order_mod_divides_exponent(Nat.2, Nat.11, Nat.10)
    multiplicative_order_mod(Nat.2, Nat.11).divides(Nat.10)
    divides_lte(multiplicative_order_mod(Nat.2, Nat.11), Nat.10)
    if Nat.10 = Nat.0 {
        lt_zero_ten
        Nat.0 < Nat.10
        Nat.0 < Nat.0
        lt_not_ref(Nat.0)
        false
    }
    multiplicative_order_mod(Nat.2, Nat.11) <= Nat.10
    multiplicative_order_mod_positive(Nat.2, Nat.11)
    Nat.0 < multiplicative_order_mod(Nat.2, Nat.11)
    trichotomy(multiplicative_order_mod(Nat.2, Nat.11), Nat.10)
    if multiplicative_order_mod(Nat.2, Nat.11) < Nat.10 {
        lt_suc_right(multiplicative_order_mod(Nat.2, Nat.11), Nat.9)
        if multiplicative_order_mod(Nat.2, Nat.11) = Nat.9 {
            multiplicative_order_mod(Nat.2, Nat.11).divides(Nat.10)
            Nat.9.divides(Nat.10)
            not_nine_divides_ten
            not Nat.9.divides(Nat.10)
            false
        }
        multiplicative_order_mod(Nat.2, Nat.11) < Nat.9
        lt_suc_right(multiplicative_order_mod(Nat.2, Nat.11), Nat.8)
        if multiplicative_order_mod(Nat.2, Nat.11) = Nat.8 {
            multiplicative_order_mod(Nat.2, Nat.11).divides(Nat.10)
            Nat.8.divides(Nat.10)
            not_eight_divides_ten
            not Nat.8.divides(Nat.10)
            false
        }
        multiplicative_order_mod(Nat.2, Nat.11) < Nat.8
        lt_suc_right(multiplicative_order_mod(Nat.2, Nat.11), Nat.7)
        if multiplicative_order_mod(Nat.2, Nat.11) = Nat.7 {
            multiplicative_order_mod(Nat.2, Nat.11).divides(Nat.10)
            Nat.7.divides(Nat.10)
            not_seven_divides_ten
            not Nat.7.divides(Nat.10)
            false
        }
        multiplicative_order_mod(Nat.2, Nat.11) < Nat.7
        lt_suc_right(multiplicative_order_mod(Nat.2, Nat.11), Nat.6)
        if multiplicative_order_mod(Nat.2, Nat.11) = Nat.6 {
            multiplicative_order_mod(Nat.2, Nat.11).divides(Nat.10)
            Nat.6.divides(Nat.10)
            not_six_divides_ten
            not Nat.6.divides(Nat.10)
            false
        }
        multiplicative_order_mod(Nat.2, Nat.11) < Nat.6
        lt_suc_right(multiplicative_order_mod(Nat.2, Nat.11), Nat.5)
        if multiplicative_order_mod(Nat.2, Nat.11) = Nat.5 {
            multiplicative_order_mod_is_order(Nat.2, Nat.11)
            is_multiplicative_order_mod(Nat.2, Nat.11, multiplicative_order_mod(Nat.2, Nat.11))
            is_multiplicative_order_mod(Nat.2, Nat.11, Nat.5)
            multiplicative_order_pow_congr_one(Nat.2, Nat.11, Nat.5)
            Nat.2.pow(Nat.5).congr_mod(Nat.1, Nat.11)
            not_congr_two_pow_five_mod_eleven
            false
        }
        multiplicative_order_mod(Nat.2, Nat.11) < Nat.5
        lt_suc_right(multiplicative_order_mod(Nat.2, Nat.11), Nat.4)
        if multiplicative_order_mod(Nat.2, Nat.11) = Nat.4 {
            multiplicative_order_mod(Nat.2, Nat.11).divides(Nat.10)
            Nat.4.divides(Nat.10)
            not_four_divides_ten
            not Nat.4.divides(Nat.10)
            false
        }
        multiplicative_order_mod(Nat.2, Nat.11) < Nat.4
        lt_suc_right(multiplicative_order_mod(Nat.2, Nat.11), Nat.3)
        if multiplicative_order_mod(Nat.2, Nat.11) = Nat.3 {
            multiplicative_order_mod(Nat.2, Nat.11).divides(Nat.10)
            Nat.3.divides(Nat.10)
            not_three_divides_ten
            not Nat.3.divides(Nat.10)
            false
        }
        multiplicative_order_mod(Nat.2, Nat.11) < Nat.3
        lt_suc_right(multiplicative_order_mod(Nat.2, Nat.11), Nat.2)
        if multiplicative_order_mod(Nat.2, Nat.11) = Nat.2 {
            multiplicative_order_mod_is_order(Nat.2, Nat.11)
            is_multiplicative_order_mod(Nat.2, Nat.11, multiplicative_order_mod(Nat.2, Nat.11))
            is_multiplicative_order_mod(Nat.2, Nat.11, Nat.2)
            multiplicative_order_pow_congr_one(Nat.2, Nat.11, Nat.2)
            Nat.2.pow(Nat.2).congr_mod(Nat.1, Nat.11)
            not_congr_two_pow_two_mod_eleven
            false
        }
        multiplicative_order_mod(Nat.2, Nat.11) < Nat.2
        lt_suc_right(multiplicative_order_mod(Nat.2, Nat.11), Nat.1)
        if multiplicative_order_mod(Nat.2, Nat.11) = Nat.1 {
            multiplicative_order_mod_is_order(Nat.2, Nat.11)
            is_multiplicative_order_mod(Nat.2, Nat.11, multiplicative_order_mod(Nat.2, Nat.11))
            is_multiplicative_order_mod(Nat.2, Nat.11, Nat.1)
            multiplicative_order_pow_congr_one(Nat.2, Nat.11, Nat.1)
            Nat.2.pow(Nat.1).congr_mod(Nat.1, Nat.11)
            not_congr_two_pow_one_mod_eleven
            false
        }
        multiplicative_order_mod(Nat.2, Nat.11) < Nat.1
        lt_suc_right(multiplicative_order_mod(Nat.2, Nat.11), Nat.0)
        if multiplicative_order_mod(Nat.2, Nat.11) = Nat.0 {
            Nat.0 < multiplicative_order_mod(Nat.2, Nat.11)
            Nat.0 < Nat.0
            lt_not_ref(Nat.0)
            false
        }
        multiplicative_order_mod(Nat.2, Nat.11) < Nat.0
        not_lt_zero(multiplicative_order_mod(Nat.2, Nat.11))
        false
    }
    if Nat.10 < multiplicative_order_mod(Nat.2, Nat.11) {
        lte_imp_not_lt(multiplicative_order_mod(Nat.2, Nat.11), Nat.10)
        not Nat.10 < multiplicative_order_mod(Nat.2, Nat.11)
        false
    }
    multiplicative_order_mod(Nat.2, Nat.11) = Nat.10
}

/// `2` is a primitive root modulo `11`.
theorem two_is_primitive_root_mod_eleven {
    is_primitive_root_mod(Nat.2, Nat.11)
} by {
    eleven_is_prime
    Nat.11.is_prime
    two_coprime_mod_eleven
    Nat.2.coprime(Nat.11)
    order_two_mod_eleven
    multiplicative_order_mod(Nat.2, Nat.11) = Nat.10
    sub_eleven_one
    Nat.11 - Nat.1 = Nat.10
    multiplicative_order_mod(Nat.2, Nat.11) = Nat.11 - Nat.1
    Nat.11.is_prime and Nat.2.coprime(Nat.11) and
        multiplicative_order_mod(Nat.2, Nat.11) = Nat.11 - Nat.1
    is_primitive_root_mod(Nat.2, Nat.11) =
        (Nat.11.is_prime and Nat.2.coprime(Nat.11) and
            multiplicative_order_mod(Nat.2, Nat.11) = Nat.11 - Nat.1)
    is_primitive_root_mod(Nat.2, Nat.11)
}

// ---------------------------------------------------------------------------
// The primitive root `2` modulo `13`.
// ---------------------------------------------------------------------------

/// `1 < 13`.
theorem one_lt_thirteen {
    Nat.1 < Nat.13
} by {
    Nat.1 < Nat.2
    lt_imp_lt_suc(Nat.1, Nat.2)
    Nat.1 < Nat.3
    lt_imp_lt_suc(Nat.1, Nat.3)
    Nat.1 < Nat.4
    lt_imp_lt_suc(Nat.1, Nat.4)
    Nat.1 < Nat.5
    lt_imp_lt_suc(Nat.1, Nat.5)
    Nat.1 < Nat.6
    lt_imp_lt_suc(Nat.1, Nat.6)
    Nat.1 < Nat.7
    lt_imp_lt_suc(Nat.1, Nat.7)
    Nat.1 < Nat.8
    lt_imp_lt_suc(Nat.1, Nat.8)
    Nat.1 < Nat.9
    lt_imp_lt_suc(Nat.1, Nat.9)
    Nat.1 < Nat.10
    lt_imp_lt_suc(Nat.1, Nat.10)
    Nat.1 < Nat.11
    lt_imp_lt_suc(Nat.1, Nat.11)
    Nat.1 < Nat.12
    lt_imp_lt_suc(Nat.1, Nat.12)
    Nat.1 < Nat.12.suc
    Nat.12.suc = Nat.13
    Nat.1 < Nat.13
}

/// `2 < 13`.
theorem two_lt_thirteen {
    Nat.2 < Nat.13
} by {
    lt_suc(Nat.2)
    Nat.2 < Nat.3
    lt_imp_lt_suc(Nat.2, Nat.3)
    Nat.2 < Nat.4
    lt_imp_lt_suc(Nat.2, Nat.4)
    Nat.2 < Nat.5
    lt_imp_lt_suc(Nat.2, Nat.5)
    Nat.2 < Nat.6
    lt_imp_lt_suc(Nat.2, Nat.6)
    Nat.2 < Nat.7
    lt_imp_lt_suc(Nat.2, Nat.7)
    Nat.2 < Nat.8
    lt_imp_lt_suc(Nat.2, Nat.8)
    Nat.2 < Nat.9
    lt_imp_lt_suc(Nat.2, Nat.9)
    Nat.2 < Nat.10
    lt_imp_lt_suc(Nat.2, Nat.10)
    Nat.2 < Nat.11
    lt_imp_lt_suc(Nat.2, Nat.11)
    Nat.2 < Nat.12
    lt_imp_lt_suc(Nat.2, Nat.12)
    Nat.2 < Nat.12.suc
    Nat.12.suc = Nat.13
    Nat.2 < Nat.13
}

/// `3 < 13`.
theorem three_lt_thirteen {
    Nat.3 < Nat.13
} by {
    lt_suc(Nat.3)
    Nat.3 < Nat.4
    lt_imp_lt_suc(Nat.3, Nat.4)
    Nat.3 < Nat.5
    lt_imp_lt_suc(Nat.3, Nat.5)
    Nat.3 < Nat.6
    lt_imp_lt_suc(Nat.3, Nat.6)
    Nat.3 < Nat.7
    lt_imp_lt_suc(Nat.3, Nat.7)
    Nat.3 < Nat.8
    lt_imp_lt_suc(Nat.3, Nat.8)
    Nat.3 < Nat.9
    lt_imp_lt_suc(Nat.3, Nat.9)
    Nat.3 < Nat.10
    lt_imp_lt_suc(Nat.3, Nat.10)
    Nat.3 < Nat.11
    lt_imp_lt_suc(Nat.3, Nat.11)
    Nat.3 < Nat.12
    lt_imp_lt_suc(Nat.3, Nat.12)
    Nat.3 < Nat.12.suc
    Nat.12.suc = Nat.13
    Nat.3 < Nat.13
}

/// `8 < 13`.
theorem eight_lt_thirteen {
    Nat.8 < Nat.13
} by {
    lt_suc(Nat.8)
    Nat.8 < Nat.9
    lt_imp_lt_suc(Nat.8, Nat.9)
    Nat.8 < Nat.10
    lt_imp_lt_suc(Nat.8, Nat.10)
    Nat.8 < Nat.11
    lt_imp_lt_suc(Nat.8, Nat.11)
    Nat.8 < Nat.12
    lt_imp_lt_suc(Nat.8, Nat.12)
    Nat.8 < Nat.12.suc
    Nat.12.suc = Nat.13
    Nat.8 < Nat.13
}

/// `12 < 13`.
theorem twelve_lt_thirteen {
    Nat.12 < Nat.13
} by {
    lt_suc(Nat.12)
    Nat.12 < Nat.12.suc
    Nat.12.suc = Nat.13
    Nat.12 < Nat.13
}

/// `0 < 6`.
theorem lt_zero_six {
    Nat.0 < Nat.6
} by {
    lt_suc(Nat.0)
    Nat.0 < Nat.1
    lt_imp_lt_suc(Nat.0, Nat.1)
    Nat.0 < Nat.2
    lt_imp_lt_suc(Nat.0, Nat.2)
    Nat.0 < Nat.3
    lt_imp_lt_suc(Nat.0, Nat.3)
    Nat.0 < Nat.4
    lt_imp_lt_suc(Nat.0, Nat.4)
    Nat.0 < Nat.5
    lt_imp_lt_suc(Nat.0, Nat.5)
    Nat.0 < Nat.6
}

/// `0 < 12`.
theorem lt_zero_twelve {
    Nat.0 < Nat.12
} by {
    lt_suc(Nat.0)
    Nat.0 < Nat.1
    lt_imp_lt_suc(Nat.0, Nat.1)
    Nat.0 < Nat.2
    lt_imp_lt_suc(Nat.0, Nat.2)
    Nat.0 < Nat.3
    lt_imp_lt_suc(Nat.0, Nat.3)
    Nat.0 < Nat.4
    lt_imp_lt_suc(Nat.0, Nat.4)
    Nat.0 < Nat.5
    lt_imp_lt_suc(Nat.0, Nat.5)
    Nat.0 < Nat.6
    lt_imp_lt_suc(Nat.0, Nat.6)
    Nat.0 < Nat.7
    lt_imp_lt_suc(Nat.0, Nat.7)
    Nat.0 < Nat.8
    lt_imp_lt_suc(Nat.0, Nat.8)
    Nat.0 < Nat.9
    lt_imp_lt_suc(Nat.0, Nat.9)
    Nat.0 < Nat.10
    lt_imp_lt_suc(Nat.0, Nat.10)
    Nat.0 < Nat.11
    lt_imp_lt_suc(Nat.0, Nat.11)
    Nat.0 < Nat.12
}

/// `3 < 5`.
theorem lt_three_five {
    Nat.3 < Nat.5
} by {
    lt_suc(Nat.3)
    Nat.3 < Nat.4
    lt_imp_lt_suc(Nat.3, Nat.4)
    Nat.3 < Nat.5
}

/// `6 < 7`.
theorem lt_six_seven {
    Nat.6 < Nat.7
} by {
    lt_suc(Nat.6)
    Nat.6 < Nat.7
}

/// `5 < 8`.
theorem lt_five_eight {
    Nat.5 < Nat.8
} by {
    lt_suc(Nat.5)
    Nat.5 < Nat.6
    lt_imp_lt_suc(Nat.5, Nat.6)
    Nat.5 < Nat.7
    lt_imp_lt_suc(Nat.5, Nat.7)
    Nat.5 < Nat.8
}

/// `4 < 9`.
theorem lt_four_nine {
    Nat.4 < Nat.9
} by {
    lt_suc(Nat.4)
    Nat.4 < Nat.5
    lt_imp_lt_suc(Nat.4, Nat.5)
    Nat.4 < Nat.6
    lt_imp_lt_suc(Nat.4, Nat.6)
    Nat.4 < Nat.7
    lt_imp_lt_suc(Nat.4, Nat.7)
    Nat.4 < Nat.8
    lt_imp_lt_suc(Nat.4, Nat.8)
    Nat.4 < Nat.9
}

/// `3 < 10`.
theorem lt_three_ten {
    Nat.3 < Nat.10
} by {
    lt_suc(Nat.3)
    Nat.3 < Nat.4
    lt_imp_lt_suc(Nat.3, Nat.4)
    Nat.3 < Nat.5
    lt_imp_lt_suc(Nat.3, Nat.5)
    Nat.3 < Nat.6
    lt_imp_lt_suc(Nat.3, Nat.6)
    Nat.3 < Nat.7
    lt_imp_lt_suc(Nat.3, Nat.7)
    Nat.3 < Nat.8
    lt_imp_lt_suc(Nat.3, Nat.8)
    Nat.3 < Nat.9
    lt_imp_lt_suc(Nat.3, Nat.9)
    Nat.3 < Nat.10
}

/// `2 < 11`.
theorem lt_two_eleven {
    Nat.2 < Nat.11
} by {
    lt_suc(Nat.2)
    Nat.2 < Nat.3
    lt_imp_lt_suc(Nat.2, Nat.3)
    Nat.2 < Nat.4
    lt_imp_lt_suc(Nat.2, Nat.4)
    Nat.2 < Nat.5
    lt_imp_lt_suc(Nat.2, Nat.5)
    Nat.2 < Nat.6
    lt_imp_lt_suc(Nat.2, Nat.6)
    Nat.2 < Nat.7
    lt_imp_lt_suc(Nat.2, Nat.7)
    Nat.2 < Nat.8
    lt_imp_lt_suc(Nat.2, Nat.8)
    Nat.2 < Nat.9
    lt_imp_lt_suc(Nat.2, Nat.9)
    Nat.2 < Nat.10
    lt_imp_lt_suc(Nat.2, Nat.10)
    Nat.2 < Nat.11
}

/// `2 < 5`.
theorem lt_two_five {
    Nat.2 < Nat.5
} by {
    lt_suc(Nat.2)
    Nat.2 < Nat.3
    lt_imp_lt_suc(Nat.2, Nat.3)
    Nat.2 < Nat.4
    lt_imp_lt_suc(Nat.2, Nat.4)
    Nat.2 < Nat.5
}

/// `5 < 7`.
theorem lt_five_seven {
    Nat.5 < Nat.7
} by {
    lt_suc(Nat.5)
    Nat.5 < Nat.6
    lt_imp_lt_suc(Nat.5, Nat.6)
    Nat.5 < Nat.7
}

/// `4 < 8`.
theorem lt_four_eight {
    Nat.4 < Nat.8
} by {
    lt_suc(Nat.4)
    Nat.4 < Nat.5
    lt_imp_lt_suc(Nat.4, Nat.5)
    Nat.4 < Nat.6
    lt_imp_lt_suc(Nat.4, Nat.6)
    Nat.4 < Nat.7
    lt_imp_lt_suc(Nat.4, Nat.7)
    Nat.4 < Nat.8
}

/// `3 < 9`.
theorem lt_three_nine {
    Nat.3 < Nat.9
} by {
    lt_suc(Nat.3)
    Nat.3 < Nat.4
    lt_imp_lt_suc(Nat.3, Nat.4)
    Nat.3 < Nat.5
    lt_imp_lt_suc(Nat.3, Nat.5)
    Nat.3 < Nat.6
    lt_imp_lt_suc(Nat.3, Nat.6)
    Nat.3 < Nat.7
    lt_imp_lt_suc(Nat.3, Nat.7)
    Nat.3 < Nat.8
    lt_imp_lt_suc(Nat.3, Nat.8)
    Nat.3 < Nat.9
}

/// `2 < 10`.
theorem lt_two_ten {
    Nat.2 < Nat.10
} by {
    lt_suc(Nat.2)
    Nat.2 < Nat.3
    lt_imp_lt_suc(Nat.2, Nat.3)
    Nat.2 < Nat.4
    lt_imp_lt_suc(Nat.2, Nat.4)
    Nat.2 < Nat.5
    lt_imp_lt_suc(Nat.2, Nat.5)
    Nat.2 < Nat.6
    lt_imp_lt_suc(Nat.2, Nat.6)
    Nat.2 < Nat.7
    lt_imp_lt_suc(Nat.2, Nat.7)
    Nat.2 < Nat.8
    lt_imp_lt_suc(Nat.2, Nat.8)
    Nat.2 < Nat.9
    lt_imp_lt_suc(Nat.2, Nat.9)
    Nat.2 < Nat.10
}

/// `1 + 11 = 12`.
theorem one_plus_eleven {
    Nat.1 + Nat.11 = Nat.12
} by {
    add_comm(Nat.1, Nat.11)
    Nat.1 + Nat.11 = Nat.11 + Nat.1
    eleven_plus_one
    Nat.11 + Nat.1 = Nat.12
}

/// `12 + 1 = 13`.
theorem twelve_plus_one {
    Nat.12 + Nat.1 = Nat.13
} by {
    add_one_right(Nat.12)
    Nat.12 + Nat.1 = Nat.12.suc
    Nat.12.suc = Nat.13
}

/// `11 != 1`.
theorem eleven_ne_one {
    Nat.11 != Nat.1
} by {
    lt_ne(Nat.1, Nat.11, Nat.10)
    one_plus_ten
    Nat.1 + Nat.10 = Nat.11
    Nat.10 != Nat.0
    Nat.1 != Nat.11
    Nat.11 != Nat.1
}

/// `12 != 1`.
theorem twelve_ne_one {
    Nat.12 != Nat.1
} by {
    lt_ne(Nat.1, Nat.12, Nat.11)
    one_plus_eleven
    Nat.1 + Nat.11 = Nat.12
    Nat.11 != Nat.0
    Nat.1 != Nat.12
    Nat.12 != Nat.1
}

/// `13 - 10 = 3`.
theorem sub_thirteen_ten {
    Nat.13 - Nat.10 = Nat.3
} by {
    three_plus_ten
    Nat.3 + Nat.10 = Nat.13
    add_imp_sub(Nat.3, Nat.10, Nat.13)
    Nat.13 - Nat.10 = Nat.3
}

/// `13 - 7 = 6`.
theorem sub_thirteen_seven {
    Nat.13 - Nat.7 = Nat.6
} by {
    six_plus_seven
    Nat.6 + Nat.7 = Nat.13
    add_imp_sub(Nat.6, Nat.7, Nat.13)
    Nat.13 - Nat.7 = Nat.6
}

/// `13 - 8 = 5`.
theorem sub_thirteen_eight {
    Nat.13 - Nat.8 = Nat.5
} by {
    five_plus_eight
    Nat.5 + Nat.8 = Nat.13
    add_imp_sub(Nat.5, Nat.8, Nat.13)
    Nat.13 - Nat.8 = Nat.5
}

/// `13 - 9 = 4`.
theorem sub_thirteen_nine {
    Nat.13 - Nat.9 = Nat.4
} by {
    four_plus_nine
    Nat.4 + Nat.9 = Nat.13
    add_imp_sub(Nat.4, Nat.9, Nat.13)
    Nat.13 - Nat.9 = Nat.4
}

/// `13 - 11 = 2`.
theorem sub_thirteen_eleven {
    Nat.13 - Nat.11 = Nat.2
} by {
    nat_add_11_2
    Nat.11 + Nat.2 = Nat.13
    add_imp_sub_left(Nat.11, Nat.2, Nat.13)
    Nat.13 - Nat.11 = Nat.2
}

/// `13 - 1 = 12`.
theorem sub_thirteen_one {
    Nat.13 - Nat.1 = Nat.12
} by {
    twelve_plus_one
    Nat.12 + Nat.1 = Nat.13
    add_imp_sub(Nat.12, Nat.1, Nat.13)
    Nat.13 - Nat.1 = Nat.12
}

/// `12 - 10 = 2`.
theorem sub_twelve_ten {
    Nat.12 - Nat.10 = Nat.2
} by {
    ten_plus_two
    Nat.10 + Nat.2 = Nat.12
    add_imp_sub(Nat.2, Nat.10, Nat.12)
    Nat.12 - Nat.10 = Nat.2
}

/// `12 - 7 = 5`.
theorem sub_twelve_seven {
    Nat.12 - Nat.7 = Nat.5
} by {
    seven_plus_five
    Nat.7 + Nat.5 = Nat.12
    add_imp_sub(Nat.5, Nat.7, Nat.12)
    Nat.12 - Nat.7 = Nat.5
}

/// `12 - 8 = 4`.
theorem sub_twelve_eight {
    Nat.12 - Nat.8 = Nat.4
} by {
    eight_plus_four
    Nat.8 + Nat.4 = Nat.12
    add_imp_sub(Nat.4, Nat.8, Nat.12)
    Nat.12 - Nat.8 = Nat.4
}

/// `12 - 9 = 3`.
theorem sub_twelve_nine {
    Nat.12 - Nat.9 = Nat.3
} by {
    nine_plus_three
    Nat.9 + Nat.3 = Nat.12
    add_imp_sub(Nat.3, Nat.9, Nat.12)
    Nat.12 - Nat.9 = Nat.3
}

/// `12 - 11 = 1`.
theorem sub_twelve_eleven {
    Nat.12 - Nat.11 = Nat.1
} by {
    eleven_plus_one
    Nat.11 + Nat.1 = Nat.12
    add_imp_sub(Nat.1, Nat.11, Nat.12)
    Nat.12 - Nat.11 = Nat.1
}

/// `2` divides `12`.
theorem two_divides_twelve {
    Nat.2.divides(Nat.12)
} by {
    nat_mul_2_6
    Nat.2 * Nat.6 = Nat.12
    exists(c: Nat) { Nat.2 * c = Nat.12 }
    Nat.2.divides(Nat.12)
}

/// `3` divides `12`.
theorem three_divides_twelve {
    Nat.3.divides(Nat.12)
} by {
    nat_mul_3_4
    Nat.3 * Nat.4 = Nat.12
    exists(c: Nat) { Nat.3 * c = Nat.12 }
    Nat.3.divides(Nat.12)
}

/// `4` divides `12`.
theorem four_divides_twelve {
    Nat.4.divides(Nat.12)
} by {
    nat_mul_4_3
    Nat.4 * Nat.3 = Nat.12
    exists(c: Nat) { Nat.4 * c = Nat.12 }
    Nat.4.divides(Nat.12)
}

/// `6` divides `12`.
theorem six_divides_twelve {
    Nat.6.divides(Nat.12)
} by {
    nat_mul_6_2
    Nat.6 * Nat.2 = Nat.12
    exists(c: Nat) { Nat.6 * c = Nat.12 }
    Nat.6.divides(Nat.12)
}

/// `12` divides `12`.
theorem twelve_divides_twelve {
    Nat.12.divides(Nat.12)
} by {
    mul_one_right(Nat.12)
    Nat.12 * Nat.1 = Nat.12
    exists(c: Nat) { Nat.12 * c = Nat.12 }
    Nat.12.divides(Nat.12)
}

/// Two does not divide thirteen.
theorem not_two_divides_thirteen {
    not Nat.2.divides(Nat.13)
} by {
    if Nat.2.divides(Nat.13) {
        two_divides_twelve
        Nat.2.divides(Nat.12)
        divides_suc_pair_imp_one(Nat.2, Nat.12)
        Nat.2 = Nat.1
        two_ne_one
        false
    }
}

/// Three does not divide thirteen.
theorem not_three_divides_thirteen {
    not Nat.3.divides(Nat.13)
} by {
    if Nat.3.divides(Nat.13) {
        three_divides_twelve
        Nat.3.divides(Nat.12)
        divides_suc_pair_imp_one(Nat.3, Nat.12)
        Nat.3 = Nat.1
        three_ne_one
        false
    }
}

/// Four does not divide thirteen.
theorem not_four_divides_thirteen {
    not Nat.4.divides(Nat.13)
} by {
    if Nat.4.divides(Nat.13) {
        four_divides_twelve
        Nat.4.divides(Nat.12)
        divides_suc_pair_imp_one(Nat.4, Nat.12)
        Nat.4 = Nat.1
        four_ne_one
        false
    }
}

/// Six does not divide thirteen.
theorem not_six_divides_thirteen {
    not Nat.6.divides(Nat.13)
} by {
    if Nat.6.divides(Nat.13) {
        six_divides_twelve
        Nat.6.divides(Nat.12)
        divides_suc_pair_imp_one(Nat.6, Nat.12)
        Nat.6 = Nat.1
        six_ne_one
        false
    }
}

/// Five does not divide thirteen.
theorem not_five_divides_thirteen {
    not Nat.5.divides(Nat.13)
} by {
    if Nat.5.divides(Nat.13) {
        five_divides_ten
        Nat.5.divides(Nat.10)
        divides_sub(Nat.13, Nat.10, Nat.5)
        Nat.5.divides(Nat.13 - Nat.10)
        sub_thirteen_ten
        Nat.13 - Nat.10 = Nat.3
        Nat.5.divides(Nat.3)
        lt_zero_three
        Nat.0 < Nat.3
        lt_three_five
        Nat.3 < Nat.5
        not_divides_of_lt(Nat.5, Nat.3)
        not Nat.5.divides(Nat.3)
        false
    }
}

/// Seven does not divide thirteen.
theorem not_seven_divides_thirteen {
    not Nat.7.divides(Nat.13)
} by {
    if Nat.7.divides(Nat.13) {
        divides_self(Nat.7)
        Nat.7.divides(Nat.7)
        divides_sub(Nat.13, Nat.7, Nat.7)
        Nat.7.divides(Nat.13 - Nat.7)
        sub_thirteen_seven
        Nat.13 - Nat.7 = Nat.6
        Nat.7.divides(Nat.6)
        lt_zero_six
        Nat.0 < Nat.6
        lt_six_seven
        Nat.6 < Nat.7
        not_divides_of_lt(Nat.7, Nat.6)
        not Nat.7.divides(Nat.6)
        false
    }
}

/// Eight does not divide thirteen.
theorem not_eight_divides_thirteen {
    not Nat.8.divides(Nat.13)
} by {
    if Nat.8.divides(Nat.13) {
        divides_self(Nat.8)
        Nat.8.divides(Nat.8)
        divides_sub(Nat.13, Nat.8, Nat.8)
        Nat.8.divides(Nat.13 - Nat.8)
        sub_thirteen_eight
        Nat.13 - Nat.8 = Nat.5
        Nat.8.divides(Nat.5)
        lt_zero_five
        Nat.0 < Nat.5
        lt_five_eight
        Nat.5 < Nat.8
        not_divides_of_lt(Nat.8, Nat.5)
        not Nat.8.divides(Nat.5)
        false
    }
}

/// Nine does not divide thirteen.
theorem not_nine_divides_thirteen {
    not Nat.9.divides(Nat.13)
} by {
    if Nat.9.divides(Nat.13) {
        divides_self(Nat.9)
        Nat.9.divides(Nat.9)
        divides_sub(Nat.13, Nat.9, Nat.9)
        Nat.9.divides(Nat.13 - Nat.9)
        sub_thirteen_nine
        Nat.13 - Nat.9 = Nat.4
        Nat.9.divides(Nat.4)
        lt_zero_four
        Nat.0 < Nat.4
        lt_four_nine
        Nat.4 < Nat.9
        not_divides_of_lt(Nat.9, Nat.4)
        not Nat.9.divides(Nat.4)
        false
    }
}

/// Ten does not divide thirteen.
theorem not_ten_divides_thirteen {
    not Nat.10.divides(Nat.13)
} by {
    if Nat.10.divides(Nat.13) {
        mul_one_right(Nat.10)
        Nat.10 * Nat.1 = Nat.10
        exists(c: Nat) { Nat.10 * c = Nat.10 }
        Nat.10.divides(Nat.10)
        divides_sub(Nat.13, Nat.10, Nat.10)
        Nat.10.divides(Nat.13 - Nat.10)
        sub_thirteen_ten
        Nat.13 - Nat.10 = Nat.3
        Nat.10.divides(Nat.3)
        lt_zero_three
        Nat.0 < Nat.3
        lt_three_ten
        Nat.3 < Nat.10
        not_divides_of_lt(Nat.10, Nat.3)
        not Nat.10.divides(Nat.3)
        false
    }
}

/// Eleven does not divide thirteen.
theorem not_eleven_divides_thirteen {
    not Nat.11.divides(Nat.13)
} by {
    if Nat.11.divides(Nat.13) {
        mul_one_right(Nat.11)
        Nat.11 * Nat.1 = Nat.11
        exists(c: Nat) { Nat.11 * c = Nat.11 }
        Nat.11.divides(Nat.11)
        divides_sub(Nat.13, Nat.11, Nat.11)
        Nat.11.divides(Nat.13 - Nat.11)
        sub_thirteen_eleven
        Nat.13 - Nat.11 = Nat.2
        Nat.11.divides(Nat.2)
        lt_zero_two
        Nat.0 < Nat.2
        lt_two_eleven
        Nat.2 < Nat.11
        not_divides_of_lt(Nat.11, Nat.2)
        not Nat.11.divides(Nat.2)
        false
    }
}

/// Twelve does not divide thirteen.
theorem not_twelve_divides_thirteen {
    not Nat.12.divides(Nat.13)
} by {
    if Nat.12.divides(Nat.13) {
        twelve_divides_twelve
        Nat.12.divides(Nat.12)
        divides_suc_pair_imp_one(Nat.12, Nat.12)
        Nat.12 = Nat.1
        twelve_ne_one
        false
    }
}

/// Five does not divide twelve.
theorem not_five_divides_twelve {
    not Nat.5.divides(Nat.12)
} by {
    if Nat.5.divides(Nat.12) {
        five_divides_ten
        Nat.5.divides(Nat.10)
        divides_sub(Nat.12, Nat.10, Nat.5)
        Nat.5.divides(Nat.12 - Nat.10)
        sub_twelve_ten
        Nat.12 - Nat.10 = Nat.2
        Nat.5.divides(Nat.2)
        lt_zero_two
        Nat.0 < Nat.2
        lt_two_five
        Nat.2 < Nat.5
        not_divides_of_lt(Nat.5, Nat.2)
        not Nat.5.divides(Nat.2)
        false
    }
}

/// Seven does not divide twelve.
theorem not_seven_divides_twelve {
    not Nat.7.divides(Nat.12)
} by {
    if Nat.7.divides(Nat.12) {
        divides_self(Nat.7)
        Nat.7.divides(Nat.7)
        divides_sub(Nat.12, Nat.7, Nat.7)
        Nat.7.divides(Nat.12 - Nat.7)
        sub_twelve_seven
        Nat.12 - Nat.7 = Nat.5
        Nat.7.divides(Nat.5)
        lt_zero_five
        Nat.0 < Nat.5
        lt_five_seven
        Nat.5 < Nat.7
        not_divides_of_lt(Nat.7, Nat.5)
        not Nat.7.divides(Nat.5)
        false
    }
}

/// Eight does not divide twelve.
theorem not_eight_divides_twelve {
    not Nat.8.divides(Nat.12)
} by {
    if Nat.8.divides(Nat.12) {
        divides_self(Nat.8)
        Nat.8.divides(Nat.8)
        divides_sub(Nat.12, Nat.8, Nat.8)
        Nat.8.divides(Nat.12 - Nat.8)
        sub_twelve_eight
        Nat.12 - Nat.8 = Nat.4
        Nat.8.divides(Nat.4)
        lt_zero_four
        Nat.0 < Nat.4
        lt_four_eight
        Nat.4 < Nat.8
        not_divides_of_lt(Nat.8, Nat.4)
        not Nat.8.divides(Nat.4)
        false
    }
}

/// Nine does not divide twelve.
theorem not_nine_divides_twelve {
    not Nat.9.divides(Nat.12)
} by {
    if Nat.9.divides(Nat.12) {
        divides_self(Nat.9)
        Nat.9.divides(Nat.9)
        divides_sub(Nat.12, Nat.9, Nat.9)
        Nat.9.divides(Nat.12 - Nat.9)
        sub_twelve_nine
        Nat.12 - Nat.9 = Nat.3
        Nat.9.divides(Nat.3)
        lt_zero_three
        Nat.0 < Nat.3
        lt_three_nine
        Nat.3 < Nat.9
        not_divides_of_lt(Nat.9, Nat.3)
        not Nat.9.divides(Nat.3)
        false
    }
}

/// Ten does not divide twelve.
theorem not_ten_divides_twelve {
    not Nat.10.divides(Nat.12)
} by {
    if Nat.10.divides(Nat.12) {
        mul_one_right(Nat.10)
        Nat.10 * Nat.1 = Nat.10
        exists(c: Nat) { Nat.10 * c = Nat.10 }
        Nat.10.divides(Nat.10)
        divides_sub(Nat.12, Nat.10, Nat.10)
        Nat.10.divides(Nat.12 - Nat.10)
        sub_twelve_ten
        Nat.12 - Nat.10 = Nat.2
        Nat.10.divides(Nat.2)
        lt_zero_two
        Nat.0 < Nat.2
        lt_two_ten
        Nat.2 < Nat.10
        not_divides_of_lt(Nat.10, Nat.2)
        not Nat.10.divides(Nat.2)
        false
    }
}

/// Eleven does not divide twelve.
theorem not_eleven_divides_twelve {
    not Nat.11.divides(Nat.12)
} by {
    if Nat.11.divides(Nat.12) {
        mul_one_right(Nat.11)
        Nat.11 * Nat.1 = Nat.11
        exists(c: Nat) { Nat.11 * c = Nat.11 }
        Nat.11.divides(Nat.11)
        divides_sub(Nat.12, Nat.11, Nat.11)
        Nat.11.divides(Nat.12 - Nat.11)
        sub_twelve_eleven
        Nat.12 - Nat.11 = Nat.1
        Nat.11.divides(Nat.1)
        nat_divides_one_imp_one(Nat.11)
        Nat.11 = Nat.1
        eleven_ne_one
        false
    }
}

/// `1 < k` and `k < 12` forces `k` into `{2, ..., 11}`.
theorem k_range_two_11(k: Nat) {
    Nat.1 < k and k < Nat.12 implies (k = Nat.2 or k = Nat.3 or k = Nat.4 or k = Nat.5 or k = Nat.6 or k = Nat.7 or k = Nat.8 or k = Nat.9 or k = Nat.10 or k = Nat.11)
} by {
    if Nat.1 < k and k < Nat.12 {
        Nat.11.suc = Nat.12
        lt_suc_right(k, Nat.11)
        k = Nat.11 or k < Nat.11
        if k < Nat.11 {
            k_range_two_ten(k)
            k = Nat.2 or k = Nat.3 or k = Nat.4 or k = Nat.5 or k = Nat.6 or k = Nat.7 or k = Nat.8 or k = Nat.9 or k = Nat.10
            k = Nat.2 or k = Nat.3 or k = Nat.4 or k = Nat.5 or k = Nat.6 or k = Nat.7 or k = Nat.8 or k = Nat.9 or k = Nat.10 or k = Nat.11
        } else {
            k = Nat.11
            k = Nat.2 or k = Nat.3 or k = Nat.4 or k = Nat.5 or k = Nat.6 or k = Nat.7 or k = Nat.8 or k = Nat.9 or k = Nat.10 or k = Nat.11
        }
    }
}

/// `1 < k` and `k < 13` forces `k` into `{2, ..., 12}`.
theorem k_range_two_12(k: Nat) {
    Nat.1 < k and k < Nat.13 implies (k = Nat.2 or k = Nat.3 or k = Nat.4 or k = Nat.5 or k = Nat.6 or k = Nat.7 or k = Nat.8 or k = Nat.9 or k = Nat.10 or k = Nat.11 or k = Nat.12)
} by {
    if Nat.1 < k and k < Nat.13 {
        Nat.12.suc = Nat.13
        lt_suc_right(k, Nat.12)
        k = Nat.12 or k < Nat.12
        if k < Nat.12 {
            k_range_two_11(k)
            k = Nat.2 or k = Nat.3 or k = Nat.4 or k = Nat.5 or k = Nat.6 or k = Nat.7 or k = Nat.8 or k = Nat.9 or k = Nat.10 or k = Nat.11
            k = Nat.2 or k = Nat.3 or k = Nat.4 or k = Nat.5 or k = Nat.6 or k = Nat.7 or k = Nat.8 or k = Nat.9 or k = Nat.10 or k = Nat.11 or k = Nat.12
        } else {
            k = Nat.12
            k = Nat.2 or k = Nat.3 or k = Nat.4 or k = Nat.5 or k = Nat.6 or k = Nat.7 or k = Nat.8 or k = Nat.9 or k = Nat.10 or k = Nat.11 or k = Nat.12
        }
    }
}

/// No number strictly between 1 and 13 divides 13 (by cases).
theorem thirteen_not_divides_by_case(k: Nat) {
    Nat.1 < k and k < Nat.13 and (k = Nat.2 or k = Nat.3 or k = Nat.4 or k = Nat.5 or k = Nat.6 or k = Nat.7 or k = Nat.8 or k = Nat.9 or k = Nat.10 or k = Nat.11 or k = Nat.12)
        implies not k.divides(Nat.13)
} by {
    if Nat.1 < k and k < Nat.13 and (k = Nat.2 or k = Nat.3 or k = Nat.4 or k = Nat.5 or k = Nat.6 or k = Nat.7 or k = Nat.8 or k = Nat.9 or k = Nat.10 or k = Nat.11 or k = Nat.12) {
        if k = Nat.2 {
            not_two_divides_thirteen
            not Nat.2.divides(Nat.13)
            not k.divides(Nat.13)
        } else {
            (k = Nat.3 or k = Nat.4 or k = Nat.5 or k = Nat.6 or k = Nat.7 or k = Nat.8 or k = Nat.9 or k = Nat.10 or k = Nat.11 or k = Nat.12)
        if k = Nat.3 {
            not_three_divides_thirteen
            not Nat.3.divides(Nat.13)
            not k.divides(Nat.13)
        } else {
            (k = Nat.4 or k = Nat.5 or k = Nat.6 or k = Nat.7 or k = Nat.8 or k = Nat.9 or k = Nat.10 or k = Nat.11 or k = Nat.12)
        if k = Nat.4 {
            not_four_divides_thirteen
            not Nat.4.divides(Nat.13)
            not k.divides(Nat.13)
        } else {
            (k = Nat.5 or k = Nat.6 or k = Nat.7 or k = Nat.8 or k = Nat.9 or k = Nat.10 or k = Nat.11 or k = Nat.12)
        if k = Nat.5 {
            not_five_divides_thirteen
            not Nat.5.divides(Nat.13)
            not k.divides(Nat.13)
        } else {
            (k = Nat.6 or k = Nat.7 or k = Nat.8 or k = Nat.9 or k = Nat.10 or k = Nat.11 or k = Nat.12)
        if k = Nat.6 {
            not_six_divides_thirteen
            not Nat.6.divides(Nat.13)
            not k.divides(Nat.13)
        } else {
            (k = Nat.7 or k = Nat.8 or k = Nat.9 or k = Nat.10 or k = Nat.11 or k = Nat.12)
        if k = Nat.7 {
            not_seven_divides_thirteen
            not Nat.7.divides(Nat.13)
            not k.divides(Nat.13)
        } else {
            (k = Nat.8 or k = Nat.9 or k = Nat.10 or k = Nat.11 or k = Nat.12)
        if k = Nat.8 {
            not_eight_divides_thirteen
            not Nat.8.divides(Nat.13)
            not k.divides(Nat.13)
        } else {
            (k = Nat.9 or k = Nat.10 or k = Nat.11 or k = Nat.12)
        if k = Nat.9 {
            not_nine_divides_thirteen
            not Nat.9.divides(Nat.13)
            not k.divides(Nat.13)
        } else {
            (k = Nat.10 or k = Nat.11 or k = Nat.12)
        if k = Nat.10 {
            not_ten_divides_thirteen
            not Nat.10.divides(Nat.13)
            not k.divides(Nat.13)
        } else {
            (k = Nat.11 or k = Nat.12)
        if k = Nat.11 {
            not_eleven_divides_thirteen
            not Nat.11.divides(Nat.13)
            not k.divides(Nat.13)
        } else {
            k = Nat.12
            not_twelve_divides_thirteen
            not Nat.12.divides(Nat.13)
            not k.divides(Nat.13)
        }
        }
        }
        }
        }
        }
        }
        }
        }
        }
    }
}

/// No number strictly between 1 and 13 divides 13.
theorem thirteen_no_proper_divisor(k: Nat) {
    Nat.1 < k and k < Nat.13 implies not k.divides(Nat.13)
} by {
    if Nat.1 < k and k < Nat.13 {
        k_range_two_12(k)
        k = Nat.2 or k = Nat.3 or k = Nat.4 or k = Nat.5 or k = Nat.6 or k = Nat.7 or k = Nat.8 or k = Nat.9 or k = Nat.10 or k = Nat.11 or k = Nat.12
        thirteen_not_divides_by_case(k)
        not k.divides(Nat.13)
    }
}

/// Thirteen is prime.
theorem thirteen_is_prime {
    Nat.13.is_prime
} by {
    one_lt_thirteen
    Nat.1 < Nat.13
    forall(k: Nat) {
        thirteen_no_proper_divisor(k)
    }
    no_proper_divisor_imp_prime(Nat.13)
}

/// `2` is coprime to `13`.
theorem two_coprime_mod_thirteen {
    Nat.2.coprime(Nat.13)
} by {
    thirteen_is_prime
    Nat.13.is_prime
    Nat.1 <= Nat.2
    two_lt_thirteen
    Nat.2 < Nat.13
    coprime_below_prime(Nat.13, Nat.2)
    Nat.2.coprime(Nat.13)
}

/// `2^4 = 16`.
theorem pow_two_four {
    Nat.2.pow(Nat.4) = Nat.16
} by {
    exp_add(Nat.2, Nat.2, Nat.2)
    Nat.2.pow(Nat.2 + Nat.2) = Nat.2.pow(Nat.2) * Nat.2.pow(Nat.2)
    two_plus_two
    Nat.2 + Nat.2 = Nat.4
    Nat.2.pow(Nat.4) = Nat.2.pow(Nat.2) * Nat.2.pow(Nat.2)
    pow_two_two
    Nat.2.pow(Nat.2) = Nat.4
    Nat.2.pow(Nat.4) = Nat.4 * Nat.4
    nat_mul_4_4
    Nat.4 * Nat.4 = Nat.16
    Nat.2.pow(Nat.4) = Nat.16
}

/// `13 + 3 = 16`.
theorem nat_add_13_3 {
    Nat.13 + Nat.3 = Nat.16
} by {
    Nat.13 = Nat.1.read(Nat.3)
    read_add_single(Nat.1, Nat.3, Nat.3)
    Nat.1.read(Nat.3) + Nat.3 = Nat.1.read(Nat.3 + Nat.3)
    three_plus_three
    Nat.3 + Nat.3 = Nat.6
    Nat.1.read(Nat.6) = Nat.16
}

/// `16 ≡ 3 (mod 13)`.
theorem congr_sixteen_mod_thirteen {
    Nat.16.congr_mod(Nat.3, Nat.13)
} by {
    mul_one_left(Nat.13)
    Nat.1 * Nat.13 = Nat.13
    nat_add_13_3
    Nat.13 + Nat.3 = Nat.16
    Nat.1 * Nat.13 + Nat.3 = Nat.16
    three_lt_thirteen
    Nat.3 < Nat.13
    mod_of_decomp(Nat.1, Nat.3, Nat.13)
    (Nat.1 * Nat.13 + Nat.3).mod(Nat.13) = Nat.3
    Nat.16.mod(Nat.13) = Nat.3
    small_mod(Nat.3, Nat.13)
    Nat.3.mod(Nat.13) = Nat.3
    Nat.16.mod(Nat.13) = Nat.3.mod(Nat.13)
    Nat.16.congr_mod(Nat.3, Nat.13)
}

/// `2^4 ≡ 3 (mod 13)`.
theorem congr_two_pow_four_mod_thirteen {
    Nat.2.pow(Nat.4).congr_mod(Nat.3, Nat.13)
} by {
    pow_two_four
    Nat.2.pow(Nat.4) = Nat.16
    congr_sixteen_mod_thirteen
    Nat.16.congr_mod(Nat.3, Nat.13)
    Nat.2.pow(Nat.4).congr_mod(Nat.3, Nat.13)
}

/// `1 mod 13 = 1`.
theorem mod_one_mod_thirteen {
    Nat.1.mod(Nat.13) = Nat.1
} by {
    one_lt_thirteen
    Nat.1 < Nat.13
    small_mod(Nat.1, Nat.13)
}

/// `2 mod 13 = 2`.
theorem mod_two_mod_thirteen {
    Nat.2.mod(Nat.13) = Nat.2
} by {
    two_lt_thirteen
    Nat.2 < Nat.13
    small_mod(Nat.2, Nat.13)
}

/// `4 mod 13 = 4`.
theorem mod_four_mod_thirteen {
    Nat.4.mod(Nat.13) = Nat.4
} by {
    lt_suc(Nat.4)
    Nat.4 < Nat.5
    lt_imp_lt_suc(Nat.4, Nat.5)
    Nat.4 < Nat.6
    lt_imp_lt_suc(Nat.4, Nat.6)
    Nat.4 < Nat.7
    lt_imp_lt_suc(Nat.4, Nat.7)
    Nat.4 < Nat.8
    lt_imp_lt_suc(Nat.4, Nat.8)
    Nat.4 < Nat.9
    lt_imp_lt_suc(Nat.4, Nat.9)
    Nat.4 < Nat.10
    lt_imp_lt_suc(Nat.4, Nat.10)
    Nat.4 < Nat.11
    lt_imp_lt_suc(Nat.4, Nat.11)
    Nat.4 < Nat.12
    lt_imp_lt_suc(Nat.4, Nat.12)
    Nat.4 < Nat.13
    small_mod(Nat.4, Nat.13)
}

/// `8 mod 13 = 8`.
theorem mod_eight_mod_thirteen {
    Nat.8.mod(Nat.13) = Nat.8
} by {
    eight_lt_thirteen
    Nat.8 < Nat.13
    small_mod(Nat.8, Nat.13)
}

/// `3 ≢ 1 (mod 13)`.
theorem not_congr_three_mod_thirteen {
    not Nat.3.congr_mod(Nat.1, Nat.13)
} by {
    three_lt_thirteen
    Nat.3 < Nat.13
    small_mod(Nat.3, Nat.13)
    Nat.3.mod(Nat.13) = Nat.3
    mod_one_mod_thirteen
    Nat.1.mod(Nat.13) = Nat.1
    if Nat.3.congr_mod(Nat.1, Nat.13) {
        Nat.3.mod(Nat.13) = Nat.1.mod(Nat.13)
        Nat.3 = Nat.1
        three_ne_one
        false
    }
}

/// `12 ≢ 1 (mod 13)`.
theorem not_congr_twelve_mod_thirteen {
    not Nat.12.congr_mod(Nat.1, Nat.13)
} by {
    twelve_lt_thirteen
    Nat.12 < Nat.13
    small_mod(Nat.12, Nat.13)
    Nat.12.mod(Nat.13) = Nat.12
    mod_one_mod_thirteen
    Nat.1.mod(Nat.13) = Nat.1
    if Nat.12.congr_mod(Nat.1, Nat.13) {
        Nat.12.mod(Nat.13) = Nat.1.mod(Nat.13)
        Nat.12 = Nat.1
        twelve_ne_one
        false
    }
}

/// `2^1 ≢ 1 (mod 13)`.
theorem not_congr_two_pow_one_mod_thirteen {
    not Nat.2.pow(Nat.1).congr_mod(Nat.1, Nat.13)
} by {
    mod_two_mod_thirteen
    Nat.2.mod(Nat.13) = Nat.2
    mod_one_mod_thirteen
    Nat.1.mod(Nat.13) = Nat.1
    if Nat.2.pow(Nat.1).congr_mod(Nat.1, Nat.13) {
        pow_two_one
        Nat.2.pow(Nat.1) = Nat.2
        Nat.2.congr_mod(Nat.1, Nat.13)
        Nat.2.mod(Nat.13) = Nat.1.mod(Nat.13)
        Nat.2 = Nat.1
        two_ne_one
        false
    }
}

/// `2^2 ≢ 1 (mod 13)`.
theorem not_congr_two_pow_two_mod_thirteen {
    not Nat.2.pow(Nat.2).congr_mod(Nat.1, Nat.13)
} by {
    mod_four_mod_thirteen
    Nat.4.mod(Nat.13) = Nat.4
    mod_one_mod_thirteen
    Nat.1.mod(Nat.13) = Nat.1
    if Nat.2.pow(Nat.2).congr_mod(Nat.1, Nat.13) {
        pow_two_two
        Nat.2.pow(Nat.2) = Nat.4
        Nat.4.congr_mod(Nat.1, Nat.13)
        Nat.4.mod(Nat.13) = Nat.1.mod(Nat.13)
        Nat.4 = Nat.1
        four_ne_one
        false
    }
}

/// `2^3 ≢ 1 (mod 13)`.
theorem not_congr_two_pow_three_mod_thirteen {
    not Nat.2.pow(Nat.3).congr_mod(Nat.1, Nat.13)
} by {
    mod_eight_mod_thirteen
    Nat.8.mod(Nat.13) = Nat.8
    mod_one_mod_thirteen
    Nat.1.mod(Nat.13) = Nat.1
    if Nat.2.pow(Nat.3).congr_mod(Nat.1, Nat.13) {
        pow_two_three
        Nat.2.pow(Nat.3) = Nat.8
        Nat.8.congr_mod(Nat.1, Nat.13)
        Nat.8.mod(Nat.13) = Nat.1.mod(Nat.13)
        Nat.8 = Nat.1
        four_ne_one
        false
    }
}

/// `2^4 ≢ 1 (mod 13)`.
theorem not_congr_two_pow_four_mod_thirteen {
    not Nat.2.pow(Nat.4).congr_mod(Nat.1, Nat.13)
} by {
    congr_two_pow_four_mod_thirteen
    Nat.2.pow(Nat.4).congr_mod(Nat.3, Nat.13)
    if Nat.2.pow(Nat.4).congr_mod(Nat.1, Nat.13) {
        congr_mod_symm(Nat.2.pow(Nat.4), Nat.3, Nat.13)
        Nat.3.congr_mod(Nat.2.pow(Nat.4), Nat.13)
        congr_mod_trans(Nat.3, Nat.2.pow(Nat.4), Nat.1, Nat.13)
        Nat.3.congr_mod(Nat.1, Nat.13)
        not_congr_three_mod_thirteen
        false
    }
}

/// `2^6 ≡ 12 (mod 13)`.
theorem congr_two_pow_six_mod_thirteen {
    Nat.2.pow(Nat.6).congr_mod(Nat.12, Nat.13)
} by {
    exp_add(Nat.2, Nat.4, Nat.2)
    Nat.2.pow(Nat.4 + Nat.2) = Nat.2.pow(Nat.4) * Nat.2.pow(Nat.2)
    nat_add_4_2
    Nat.4 + Nat.2 = Nat.6
    Nat.2.pow(Nat.6) = Nat.2.pow(Nat.4) * Nat.2.pow(Nat.2)
    congr_two_pow_four_mod_thirteen
    Nat.2.pow(Nat.4).congr_mod(Nat.3, Nat.13)
    pow_two_two
    Nat.2.pow(Nat.2) = Nat.4
    congr_mod_mul(Nat.2.pow(Nat.4), Nat.2.pow(Nat.2), Nat.3, Nat.4, Nat.13)
    (Nat.2.pow(Nat.4) * Nat.2.pow(Nat.2)).congr_mod(Nat.3 * Nat.4, Nat.13)
    Nat.2.pow(Nat.6).congr_mod(Nat.3 * Nat.4, Nat.13)
    nat_mul_3_4
    Nat.3 * Nat.4 = Nat.12
    Nat.2.pow(Nat.6).congr_mod(Nat.12, Nat.13)
}

/// `2^6 ≢ 1 (mod 13)`.
theorem not_congr_two_pow_six_mod_thirteen {
    not Nat.2.pow(Nat.6).congr_mod(Nat.1, Nat.13)
} by {
    congr_two_pow_six_mod_thirteen
    Nat.2.pow(Nat.6).congr_mod(Nat.12, Nat.13)
    if Nat.2.pow(Nat.6).congr_mod(Nat.1, Nat.13) {
        congr_mod_symm(Nat.2.pow(Nat.6), Nat.12, Nat.13)
        Nat.12.congr_mod(Nat.2.pow(Nat.6), Nat.13)
        congr_mod_trans(Nat.12, Nat.2.pow(Nat.6), Nat.1, Nat.13)
        Nat.12.congr_mod(Nat.1, Nat.13)
        not_congr_twelve_mod_thirteen
        false
    }
}

/// `2` has multiplicative order `12` modulo `13`.
theorem order_two_mod_thirteen {
    multiplicative_order_mod(Nat.2, Nat.13) = Nat.12
} by {
    Nat.13 != Nat.0
    two_coprime_mod_thirteen
    Nat.2.coprime(Nat.13)
    fermat_euler(Nat.13, Nat.2)
    Nat.2.pow(Nat.13 - Nat.1).congr_mod(Nat.1, Nat.13)
    sub_thirteen_one
    Nat.13 - Nat.1 = Nat.12
    Nat.2.pow(Nat.12).congr_mod(Nat.1, Nat.13)
    multiplicative_order_mod_is_order(Nat.2, Nat.13)
    is_multiplicative_order_mod(Nat.2, Nat.13, multiplicative_order_mod(Nat.2, Nat.13))
    multiplicative_order_mod_divides_exponent(Nat.2, Nat.13, Nat.12)
    multiplicative_order_mod(Nat.2, Nat.13).divides(Nat.12)
    divides_lte(multiplicative_order_mod(Nat.2, Nat.13), Nat.12)
    if Nat.12 = Nat.0 {
        lt_zero_twelve
        Nat.0 < Nat.12
        Nat.0 < Nat.0
        lt_not_ref(Nat.0)
        false
    }
    multiplicative_order_mod(Nat.2, Nat.13) <= Nat.12
    multiplicative_order_mod_positive(Nat.2, Nat.13)
    Nat.0 < multiplicative_order_mod(Nat.2, Nat.13)
    trichotomy(multiplicative_order_mod(Nat.2, Nat.13), Nat.12)
    if multiplicative_order_mod(Nat.2, Nat.13) < Nat.12 {

        lt_suc_right(multiplicative_order_mod(Nat.2, Nat.13), Nat.11)
        if multiplicative_order_mod(Nat.2, Nat.13) = Nat.11 {
            multiplicative_order_mod(Nat.2, Nat.13).divides(Nat.12)
            Nat.11.divides(Nat.12)
            not_eleven_divides_twelve
            not Nat.11.divides(Nat.12)
            false
        }
        multiplicative_order_mod(Nat.2, Nat.13) < Nat.11
        lt_suc_right(multiplicative_order_mod(Nat.2, Nat.13), Nat.10)
        if multiplicative_order_mod(Nat.2, Nat.13) = Nat.10 {
            multiplicative_order_mod(Nat.2, Nat.13).divides(Nat.12)
            Nat.10.divides(Nat.12)
            not_ten_divides_twelve
            not Nat.10.divides(Nat.12)
            false
        }
        multiplicative_order_mod(Nat.2, Nat.13) < Nat.10
        lt_suc_right(multiplicative_order_mod(Nat.2, Nat.13), Nat.9)
        if multiplicative_order_mod(Nat.2, Nat.13) = Nat.9 {
            multiplicative_order_mod(Nat.2, Nat.13).divides(Nat.12)
            Nat.9.divides(Nat.12)
            not_nine_divides_twelve
            not Nat.9.divides(Nat.12)
            false
        }
        multiplicative_order_mod(Nat.2, Nat.13) < Nat.9
        lt_suc_right(multiplicative_order_mod(Nat.2, Nat.13), Nat.8)
        if multiplicative_order_mod(Nat.2, Nat.13) = Nat.8 {
            multiplicative_order_mod(Nat.2, Nat.13).divides(Nat.12)
            Nat.8.divides(Nat.12)
            not_eight_divides_twelve
            not Nat.8.divides(Nat.12)
            false
        }
        multiplicative_order_mod(Nat.2, Nat.13) < Nat.8
        lt_suc_right(multiplicative_order_mod(Nat.2, Nat.13), Nat.7)
        if multiplicative_order_mod(Nat.2, Nat.13) = Nat.7 {
            multiplicative_order_mod(Nat.2, Nat.13).divides(Nat.12)
            Nat.7.divides(Nat.12)
            not_seven_divides_twelve
            not Nat.7.divides(Nat.12)
            false
        }
        multiplicative_order_mod(Nat.2, Nat.13) < Nat.7
        lt_suc_right(multiplicative_order_mod(Nat.2, Nat.13), Nat.6)
        if multiplicative_order_mod(Nat.2, Nat.13) = Nat.6 {
            multiplicative_order_mod_is_order(Nat.2, Nat.13)
            is_multiplicative_order_mod(Nat.2, Nat.13, multiplicative_order_mod(Nat.2, Nat.13))
            is_multiplicative_order_mod(Nat.2, Nat.13, Nat.6)
            multiplicative_order_pow_congr_one(Nat.2, Nat.13, Nat.6)
            Nat.2.pow(Nat.6).congr_mod(Nat.1, Nat.13)
            not_congr_two_pow_six_mod_thirteen
            false
        }
        multiplicative_order_mod(Nat.2, Nat.13) < Nat.6
        lt_suc_right(multiplicative_order_mod(Nat.2, Nat.13), Nat.5)
        if multiplicative_order_mod(Nat.2, Nat.13) = Nat.5 {
            multiplicative_order_mod(Nat.2, Nat.13).divides(Nat.12)
            Nat.5.divides(Nat.12)
            not_five_divides_twelve
            not Nat.5.divides(Nat.12)
            false
        }
        multiplicative_order_mod(Nat.2, Nat.13) < Nat.5
        lt_suc_right(multiplicative_order_mod(Nat.2, Nat.13), Nat.4)
        if multiplicative_order_mod(Nat.2, Nat.13) = Nat.4 {
            multiplicative_order_mod_is_order(Nat.2, Nat.13)
            is_multiplicative_order_mod(Nat.2, Nat.13, multiplicative_order_mod(Nat.2, Nat.13))
            is_multiplicative_order_mod(Nat.2, Nat.13, Nat.4)
            multiplicative_order_pow_congr_one(Nat.2, Nat.13, Nat.4)
            Nat.2.pow(Nat.4).congr_mod(Nat.1, Nat.13)
            not_congr_two_pow_four_mod_thirteen
            false
        }
        multiplicative_order_mod(Nat.2, Nat.13) < Nat.4
        lt_suc_right(multiplicative_order_mod(Nat.2, Nat.13), Nat.3)
        if multiplicative_order_mod(Nat.2, Nat.13) = Nat.3 {
            multiplicative_order_mod_is_order(Nat.2, Nat.13)
            is_multiplicative_order_mod(Nat.2, Nat.13, multiplicative_order_mod(Nat.2, Nat.13))
            is_multiplicative_order_mod(Nat.2, Nat.13, Nat.3)
            multiplicative_order_pow_congr_one(Nat.2, Nat.13, Nat.3)
            Nat.2.pow(Nat.3).congr_mod(Nat.1, Nat.13)
            not_congr_two_pow_three_mod_thirteen
            false
        }
        multiplicative_order_mod(Nat.2, Nat.13) < Nat.3
        lt_suc_right(multiplicative_order_mod(Nat.2, Nat.13), Nat.2)
        if multiplicative_order_mod(Nat.2, Nat.13) = Nat.2 {
            multiplicative_order_mod_is_order(Nat.2, Nat.13)
            is_multiplicative_order_mod(Nat.2, Nat.13, multiplicative_order_mod(Nat.2, Nat.13))
            is_multiplicative_order_mod(Nat.2, Nat.13, Nat.2)
            multiplicative_order_pow_congr_one(Nat.2, Nat.13, Nat.2)
            Nat.2.pow(Nat.2).congr_mod(Nat.1, Nat.13)
            not_congr_two_pow_two_mod_thirteen
            false
        }
        multiplicative_order_mod(Nat.2, Nat.13) < Nat.2
        lt_suc_right(multiplicative_order_mod(Nat.2, Nat.13), Nat.1)
        if multiplicative_order_mod(Nat.2, Nat.13) = Nat.1 {
            multiplicative_order_mod_is_order(Nat.2, Nat.13)
            is_multiplicative_order_mod(Nat.2, Nat.13, multiplicative_order_mod(Nat.2, Nat.13))
            is_multiplicative_order_mod(Nat.2, Nat.13, Nat.1)
            multiplicative_order_pow_congr_one(Nat.2, Nat.13, Nat.1)
            Nat.2.pow(Nat.1).congr_mod(Nat.1, Nat.13)
            not_congr_two_pow_one_mod_thirteen
            false
        }
        multiplicative_order_mod(Nat.2, Nat.13) < Nat.1
        lt_suc_right(multiplicative_order_mod(Nat.2, Nat.13), Nat.0)
        if multiplicative_order_mod(Nat.2, Nat.13) = Nat.0 {
            Nat.0 < multiplicative_order_mod(Nat.2, Nat.13)
            Nat.0 < Nat.0
            lt_not_ref(Nat.0)
            false
        }
        multiplicative_order_mod(Nat.2, Nat.13) < Nat.0
        not_lt_zero(multiplicative_order_mod(Nat.2, Nat.13))
        false
    }
    if Nat.12 < multiplicative_order_mod(Nat.2, Nat.13) {
        lte_imp_not_lt(multiplicative_order_mod(Nat.2, Nat.13), Nat.12)
        not Nat.12 < multiplicative_order_mod(Nat.2, Nat.13)
        false
    }
    multiplicative_order_mod(Nat.2, Nat.13) = Nat.12
}

/// `2` is a primitive root modulo `13`.
theorem two_is_primitive_root_mod_thirteen {
    is_primitive_root_mod(Nat.2, Nat.13)
} by {
    thirteen_is_prime
    Nat.13.is_prime
    two_coprime_mod_thirteen
    Nat.2.coprime(Nat.13)
    order_two_mod_thirteen
    multiplicative_order_mod(Nat.2, Nat.13) = Nat.12
    sub_thirteen_one
    Nat.13 - Nat.1 = Nat.12
    multiplicative_order_mod(Nat.2, Nat.13) = Nat.13 - Nat.1
    Nat.13.is_prime and Nat.2.coprime(Nat.13) and
        multiplicative_order_mod(Nat.2, Nat.13) = Nat.13 - Nat.1
    is_primitive_root_mod(Nat.2, Nat.13) =
        (Nat.13.is_prime and Nat.2.coprime(Nat.13) and
            multiplicative_order_mod(Nat.2, Nat.13) = Nat.13 - Nat.1)
    is_primitive_root_mod(Nat.2, Nat.13)
}

// ---------------------------------------------------------------------------
// The primitive root `2` modulo `5`.
// ---------------------------------------------------------------------------

/// `2` is a primitive root modulo `5`.
theorem two_is_primitive_root_mod_five {
    is_primitive_root_mod(Nat.2, Nat.5)
} by {
    five_is_prime
    Nat.5.is_prime
    two_coprime_mod_five
    Nat.2.coprime(Nat.5)
    order_two_mod_five
    multiplicative_order_mod(Nat.2, Nat.5) = Nat.4
    Nat.5 - Nat.1 = Nat.4
    multiplicative_order_mod(Nat.2, Nat.5) = Nat.5 - Nat.1
    Nat.5.is_prime and Nat.2.coprime(Nat.5) and
        multiplicative_order_mod(Nat.2, Nat.5) = Nat.5 - Nat.1
    is_primitive_root_mod(Nat.2, Nat.5) =
        (Nat.5.is_prime and Nat.2.coprime(Nat.5) and
            multiplicative_order_mod(Nat.2, Nat.5) = Nat.5 - Nat.1)
    is_primitive_root_mod(Nat.2, Nat.5)
}

// ---------------------------------------------------------------------------
// The discrete logarithm is well defined.
// ---------------------------------------------------------------------------

/// Existence of the discrete logarithm: for a primitive root `g` modulo the
/// prime `p`, every residue `a` coprime to `p` is congruent to `g^e` for some
/// exponent `e` below `p - 1`.
theorem primitive_root_discrete_log_exists(p: Nat, g: Nat, a: Nat) {
    p.is_prime and g.coprime(p) and multiplicative_order_mod(g, p) = p - Nat.1
        and a.coprime(p)
        implies exists(e: Nat) { e < p - Nat.1 and g.pow(e).congr_mod(a, p) }
} by {
    if p.is_prime and g.coprime(p) and multiplicative_order_mod(g, p) = p - Nat.1
        and a.coprime(p) {
        Nat.1 < p
        p != Nat.0
        totient_prime(p)
        p.totient = p - Nat.1
        multiplicative_order_mod(g, p) = p.totient
        full_multiplicative_order_powers_cover_units_mod(g, p)
        powers_cover_units_mod(g, p)
        multiplicative_order_mod_is_order(g, p)
        is_multiplicative_order_mod(g, p, multiplicative_order_mod(g, p))
        reduced_power_exists_of_unit_coverage(g, p, multiplicative_order_mod(g, p), a)
        exists(k: Nat) { is_reduced_power_of_mod(a, g, p, k, multiplicative_order_mod(g, p)) }
        let k: Nat satisfy { is_reduced_power_of_mod(a, g, p, k, multiplicative_order_mod(g, p)) }
        reduced_power_index_lt(a, g, p, k, multiplicative_order_mod(g, p))
        k < multiplicative_order_mod(g, p)
        multiplicative_order_mod(g, p) = p - Nat.1
        k < p - Nat.1
        reduced_power_congr(a, g, p, k, multiplicative_order_mod(g, p))
        a.congr_mod(g.pow(k), p)
        congr_mod_symm(a, g.pow(k), p)
        g.pow(k).congr_mod(a, p)
        exists(e: Nat) { e < p - Nat.1 and g.pow(e).congr_mod(a, p) }
    }
}

/// Uniqueness of the discrete logarithm: a residue `a` coprime to `p` has at
/// most one exponent below `p - 1` whose power is congruent to it.
theorem primitive_root_discrete_log_unique(p: Nat, g: Nat, a: Nat, e: Nat, f: Nat) {
    p.is_prime and g.coprime(p) and multiplicative_order_mod(g, p) = p - Nat.1
        and a.coprime(p) and e < p - Nat.1 and f < p - Nat.1
        and g.pow(e).congr_mod(a, p) and g.pow(f).congr_mod(a, p)
        implies e = f
} by {
    if p.is_prime and g.coprime(p) and multiplicative_order_mod(g, p) = p - Nat.1
        and a.coprime(p) and e < p - Nat.1 and f < p - Nat.1
        and g.pow(e).congr_mod(a, p) and g.pow(f).congr_mod(a, p) {
        congr_mod_symm(g.pow(e), a, p)
        a.congr_mod(g.pow(e), p)
        congr_mod_symm(g.pow(f), a, p)
        a.congr_mod(g.pow(f), p)
        powers_congr_of_common_representative(g, p, a, e, f)
        g.pow(e).congr_mod(g.pow(f), p)
        primitive_root_powers_below_order_injective(p, g, e, f)
        e = f
    }
}

/// The discrete logarithm is well defined: for a primitive root `g` modulo the
/// prime `p`, every nonzero residue class `a` (coprime to `p`) has a unique
/// exponent below `p - 1` whose power is congruent to it.
theorem primitive_root_discrete_log_well_defined(p: Nat, g: Nat, a: Nat) {
    p.is_prime and g.coprime(p) and multiplicative_order_mod(g, p) = p - Nat.1
        and a.coprime(p)
        implies exists(e: Nat) { e < p - Nat.1 and g.pow(e).congr_mod(a, p) and
            forall(f: Nat) { f < p - Nat.1 and g.pow(f).congr_mod(a, p) implies f = e } }
} by {
    if p.is_prime and g.coprime(p) and multiplicative_order_mod(g, p) = p - Nat.1
        and a.coprime(p) {
        primitive_root_discrete_log_exists(p, g, a)
        let e: Nat satisfy { e < p - Nat.1 and g.pow(e).congr_mod(a, p) }
        forall(f: Nat) {
            if f < p - Nat.1 and g.pow(f).congr_mod(a, p) {
                primitive_root_discrete_log_unique(p, g, a, e, f)
                f = e
            }
        }
        exists(e2: Nat) {
            e2 < p - Nat.1 and g.pow(e2).congr_mod(a, p) and
                forall(f: Nat) { f < p - Nat.1 and g.pow(f).congr_mod(a, p) implies f = e2 }
        }
    }
}

// ---------------------------------------------------------------------------
// The number of primitive roots and their existence.
//
// The classical count — a primitive root `g` modulo the prime `p` has exactly
// `φ(p - 1)` generators — and the existence of a primitive root for every
// prime:
//
//   theorem primitive_root_count_mod_prime(p: Nat, g: Nat) {
//       p.is_prime and g.coprime(p) and multiplicative_order_mod(g, p) = p - Nat.1
//           implies p.range.filter(function(x: Nat) {
//               multiplicative_order_mod(x, p) = p - Nat.1
//           }).length = (p - Nat.1).totient
//   }
//
//   theorem primitive_root_exists_prime(p: Nat) {
//       p.is_prime implies exists(g: Nat) { is_primitive_root_mod(g, p) }
//   }
//
// are left unproved: the count needs the identity that the exponents
// `k < p - 1` with `gcd(p - 1, k) = (p - 1) / d` number `φ(d)` for every
// divisor `d` of `p - 1`, which the library does not yet have, and the
// existence statement is the classical primitive-root theorem, which is deep.
// The small case `p = 5` is verified by explicit computation in
// primitive_root_applications.ac.
// ---------------------------------------------------------------------------
