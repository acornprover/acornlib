from nat import Nat, add_one_right
from rat import Rat, add_comm, add_assoc, mul_comm, mul_assoc, distrib_left, distrib_right,
    add_zero_right, add_zero_left, mul_one_right, mul_one_left, mul_zero_right,
    mul_cancels_div, mul_cancels_div_left, mul_div_cancels, mul_cancels_right,
    add_div_distrib, mul_fractions, two_neq_zero, from_nat_add, from_nat_mul,
    pos_ne_zero, zero_lt_imp_pos, nat_lt_imp_rat_lt
from data.nat.nat_range_sum import range_sum, range_sum_suc, range_sum_zero, range_sum_one
from combinatorics import binom
from number_theory.bernoulli import bernoulli, bernoulli_zero, bernoulli_one, bernoulli_two
from algebra.ring.ring import alternating_sign, alternating_sign_zero, alternating_sign_suc,
    mul_zero_left, mul_neg_left, mul_neg_neg

numerals Nat
numerals Rat

/// The p-th power of k embedded into the rationals: the k-th summand of the
/// sum of the first n p-th powers.
define power_summand(p: Nat, k: Nat) -> Rat {
    Rat.from_nat(k.pow(p))
}

/// The sum of the p-th powers of 1, ..., n:
/// sum_{k=1}^{n} k^p, where the k = 0 term contributes zero for p >= 1.
define power_sum(p: Nat, n: Nat) -> Rat {
    range_sum(power_summand(p), n.suc)
}

/// Six is a nonzero rational.
theorem six_neq_zero {
    Rat.6 != Rat.0
} by {
    Nat.0 < Nat.6
    nat_lt_imp_rat_lt(Nat.0, Nat.6)
    Rat.from_nat(Nat.0) < Rat.from_nat(Nat.6)
    Rat.from_nat(Nat.0) = Rat.0
    Rat.0 < Rat.from_nat(Nat.6)
    zero_lt_imp_pos(Rat.from_nat(Nat.6))
    Rat.from_nat(Nat.6).is_positive
    pos_ne_zero(Rat.from_nat(Nat.6))
    Rat.from_nat(Nat.6) != Rat.0
    Rat.from_nat(Nat.6) = Rat.6
    Rat.6 != Rat.0
}

/// Four is a nonzero rational.
theorem four_neq_zero {
    Rat.4 != Rat.0
} by {
    Nat.0 < Nat.4
    nat_lt_imp_rat_lt(Nat.0, Nat.4)
    Rat.from_nat(Nat.0) < Rat.from_nat(Nat.4)
    Rat.from_nat(Nat.0) = Rat.0
    Rat.0 < Rat.from_nat(Nat.4)
    zero_lt_imp_pos(Rat.from_nat(Nat.4))
    Rat.from_nat(Nat.4).is_positive
    pos_ne_zero(Rat.from_nat(Nat.4))
    Rat.from_nat(Nat.4) != Rat.0
    Rat.from_nat(Nat.4) = Rat.4
    Rat.4 != Rat.0
}

/// The closed form n(n+1)/2 of the sum of the first powers.
define closed_first(n: Nat) -> Rat {
    Rat.from_nat(n) * (Rat.from_nat(n) + Rat.1) / Rat.2
}

/// The closed form n(n+1)(2n+1)/6 of the sum of the squares.
define closed_squares(n: Nat) -> Rat {
    Rat.from_nat(n) * (Rat.from_nat(n) + Rat.1) *
        (Rat.from_nat(Nat.2 * n) + Rat.1) / Rat.6
}

/// The closed form (n(n+1)/2)^2 of the sum of the cubes.
define closed_cubes(n: Nat) -> Rat {
    (Rat.from_nat(n) * (Rat.from_nat(n) + Rat.1) / Rat.2).pow(Nat.2)
}

/// The successor of a natural embeds as the successor value plus one.
theorem from_nat_suc(x: Nat) {
    Rat.from_nat(x.suc) = Rat.from_nat(x) + Rat.1
} by {
    add_one_right(x)
    x + Nat.1 = x.suc
    from_nat_add(x, Nat.1)
    Rat.from_nat(x) + Rat.from_nat(Nat.1) = Rat.from_nat(x + Nat.1)
    Rat.from_nat(Nat.1) = Rat.1
    Rat.from_nat(x.suc) = Rat.from_nat(x) + Rat.1
}

/// Two times two is four, in the rationals.
theorem rat_two_mul_two {
    Rat.2 * Rat.2 = Rat.4
} by {
    from_nat_mul(Nat.2, Nat.2)
    Rat.from_nat(Nat.2) * Rat.from_nat(Nat.2) = Rat.from_nat(Nat.2 * Nat.2)
    Nat.2 * Nat.2 = Nat.4
    Rat.from_nat(Nat.2) = Rat.2
    Rat.from_nat(Nat.4) = Rat.4
    Rat.2 * Rat.2 = Rat.4
}

/// Two times three is six, in the rationals.
theorem rat_two_mul_three {
    Rat.2 * Rat.3 = Rat.6
} by {
    from_nat_mul(Nat.2, Nat.3)
    Rat.from_nat(Nat.2) * Rat.from_nat(Nat.3) = Rat.from_nat(Nat.2 * Nat.3)
    Nat.2 * Nat.3 = Nat.6
    Rat.from_nat(Nat.2) = Rat.2
    Rat.from_nat(Nat.3) = Rat.3
    Rat.from_nat(Nat.6) = Rat.6
    Rat.2 * Rat.3 = Rat.6
}

/// Two times a doubled element is four times the element, in the rationals.
theorem rat_double_double(x: Rat) {
    Rat.2 * (Rat.2 * x) = Rat.4 * x
} by {
    mul_assoc(Rat.2, Rat.2, x)
    (Rat.2 * Rat.2) * x = Rat.2 * (Rat.2 * x)
    rat_two_mul_two
    Rat.2 * Rat.2 = Rat.4
    Rat.4 * x = Rat.2 * (Rat.2 * x)
    Rat.2 * (Rat.2 * x) = Rat.4 * x
}

/// One plus six is seven, in the rationals.
theorem rat_one_add_six {
    Rat.1 + Rat.6 = Rat.7
} by {
    from_nat_add(Nat.1, Nat.6)
    Rat.from_nat(Nat.1) + Rat.from_nat(Nat.6) = Rat.from_nat(Nat.1 + Nat.6)
    Nat.1 + Nat.6 = Nat.7
    Rat.from_nat(Nat.1) = Rat.1
    Rat.from_nat(Nat.6) = Rat.6
    Rat.from_nat(Nat.7) = Rat.7
    Rat.1 + Rat.6 = Rat.7
}

/// Four plus three is seven, in the rationals.
theorem rat_four_add_three {
    Rat.4 + Rat.3 = Rat.7
} by {
    from_nat_add(Nat.4, Nat.3)
    Rat.from_nat(Nat.4) + Rat.from_nat(Nat.3) = Rat.from_nat(Nat.4 + Nat.3)
    Nat.4 + Nat.3 = Nat.7
    Rat.from_nat(Nat.4) = Rat.4
    Rat.from_nat(Nat.3) = Rat.3
    Rat.from_nat(Nat.7) = Rat.7
    Rat.4 + Rat.3 = Rat.7
}

// ---------------------------------------------------------------------------
// The sum of the first powers: 1 + ... + n = n(n+1)/2.
// ---------------------------------------------------------------------------

/// The inductive step of the closed form for the sum of the first powers:
/// (x + 1)(x + 2)/2 = x(x + 1)/2 + (x + 1) in the rationals.
theorem closed_first_step(x: Rat) {
    (x + Rat.1) * (x + Rat.2) / Rat.2 =
        x * (x + Rat.1) / Rat.2 + (x + Rat.1)
} by {
    two_neq_zero
    mul_div_cancels(x + Rat.1, Rat.2)
    ((x + Rat.1) * Rat.2) / Rat.2 = x + Rat.1
    mul_comm(x + Rat.1, Rat.2)
    (x + Rat.1) * Rat.2 = Rat.2 * (x + Rat.1)
    (Rat.2 * (x + Rat.1)) / Rat.2 = x + Rat.1
    x * (x + Rat.1) / Rat.2 + (x + Rat.1) =
        x * (x + Rat.1) / Rat.2 + (Rat.2 * (x + Rat.1)) / Rat.2
    add_div_distrib(x * (x + Rat.1), Rat.2 * (x + Rat.1), Rat.2)
    (x * (x + Rat.1) + Rat.2 * (x + Rat.1)) / Rat.2 =
        x * (x + Rat.1) / Rat.2 + (Rat.2 * (x + Rat.1)) / Rat.2
    distrib_right(x, Rat.2, x + Rat.1)
    (x + Rat.2) * (x + Rat.1) = x * (x + Rat.1) + Rat.2 * (x + Rat.1)
    mul_comm(x + Rat.2, x + Rat.1)
    (x + Rat.2) * (x + Rat.1) = (x + Rat.1) * (x + Rat.2)
    (x + Rat.1) * (x + Rat.2) = x * (x + Rat.1) + Rat.2 * (x + Rat.1)
    (x + Rat.1) * (x + Rat.2) / Rat.2 =
        (x * (x + Rat.1) + Rat.2 * (x + Rat.1)) / Rat.2
    (x + Rat.1) * (x + Rat.2) / Rat.2 =
        x * (x + Rat.1) / Rat.2 + (x + Rat.1)
}

/// The closed form for the sum of the first powers satisfies its recurrence:
/// closed_first(k+1) = closed_first(k) + (k+1).
theorem closed_first_suc(k: Nat) {
    closed_first(k.suc) = closed_first(k) + Rat.from_nat(k.suc)
} by {
    from_nat_suc(k)
    Rat.from_nat(k.suc) = Rat.from_nat(k) + Rat.1
    Rat.from_nat(k.suc) + Rat.1 = Rat.from_nat(k) + Rat.1 + Rat.1
    Rat.1 + Rat.1 = Rat.2
    Rat.from_nat(k.suc) + Rat.1 = Rat.from_nat(k) + Rat.2
    closed_first(k.suc) =
        Rat.from_nat(k.suc) * (Rat.from_nat(k.suc) + Rat.1) / Rat.2
    closed_first(k.suc) =
        (Rat.from_nat(k) + Rat.1) * (Rat.from_nat(k) + Rat.2) / Rat.2
    closed_first_step(Rat.from_nat(k))
    (Rat.from_nat(k) + Rat.1) * (Rat.from_nat(k) + Rat.2) / Rat.2 =
        Rat.from_nat(k) * (Rat.from_nat(k) + Rat.1) / Rat.2 +
            (Rat.from_nat(k) + Rat.1)
    closed_first(k.suc) =
        Rat.from_nat(k) * (Rat.from_nat(k) + Rat.1) / Rat.2 +
            (Rat.from_nat(k) + Rat.1)
    closed_first(k) = Rat.from_nat(k) * (Rat.from_nat(k) + Rat.1) / Rat.2
    closed_first(k.suc) = closed_first(k) + (Rat.from_nat(k) + Rat.1)
    closed_first(k.suc) = closed_first(k) + Rat.from_nat(k.suc)
}

/// The sum of the first n positive integers is n(n+1)/2:
/// sum_{k=1}^{n} k = n(n+1)/2.
theorem sum_of_first(n: Nat) {
    power_sum(Nat.1, n) = closed_first(n)
} by {
    define p(k: Nat) -> Bool {
        power_sum(Nat.1, k) = closed_first(k)
    }
    range_sum_one(power_summand(Nat.1))
    range_sum(power_summand(Nat.1), Nat.1) = power_summand(Nat.1, Nat.0)
    power_sum(Nat.1, Nat.0) = range_sum(power_summand(Nat.1), Nat.0.suc)
    Nat.0.suc = Nat.1
    power_sum(Nat.1, Nat.0) = power_summand(Nat.1, Nat.0)
    power_summand(Nat.1, Nat.0) = Rat.from_nat(Nat.0.pow(Nat.1))
    Nat.0.pow(Nat.1) = Nat.0
    Rat.from_nat(Nat.0) = Rat.0
    power_sum(Nat.1, Nat.0) = Rat.0
    closed_first(Nat.0) = Rat.from_nat(Nat.0) * (Rat.from_nat(Nat.0) + Rat.1) / Rat.2
    Rat.from_nat(Nat.0) = Rat.0
    Rat.0 * (Rat.0 + Rat.1) / Rat.2 = Rat.0
    closed_first(Nat.0) = Rat.0
    power_sum(Nat.1, Nat.0) = closed_first(Nat.0)
    p(Nat.0)
    forall(k: Nat) {
        if p(k) {
            power_sum(Nat.1, k) = closed_first(k)
            range_sum_suc(power_summand(Nat.1), k.suc)
            range_sum(power_summand(Nat.1), k.suc.suc) =
                range_sum(power_summand(Nat.1), k.suc) +
                    power_summand(Nat.1, k.suc)
            power_sum(Nat.1, k.suc) =
                range_sum(power_summand(Nat.1), k.suc.suc)
            power_sum(Nat.1, k) = range_sum(power_summand(Nat.1), k.suc)
            power_summand(Nat.1, k.suc) = Rat.from_nat(k.suc.pow(Nat.1))
            k.suc.pow(Nat.1) = k.suc
            power_summand(Nat.1, k.suc) = Rat.from_nat(k.suc)
            power_sum(Nat.1, k.suc) = power_sum(Nat.1, k) + Rat.from_nat(k.suc)
            power_sum(Nat.1, k.suc) = closed_first(k) + Rat.from_nat(k.suc)
            closed_first_suc(k)
            closed_first(k.suc) = closed_first(k) + Rat.from_nat(k.suc)
            power_sum(Nat.1, k.suc) = closed_first(k.suc)
            p(k.suc)
        }
    }
    p(Nat.0) and forall(k: Nat) {
        p(k) implies p(k.suc)
    }
    Nat.induction(p)
    p(n)
}

// ---------------------------------------------------------------------------
// The sum of the squares: 1^2 + ... + n^2 = n(n+1)(2n+1)/6.
// ---------------------------------------------------------------------------

/// The inner polynomial identity behind the closed form of the sum of
/// squares: x(2x+1) + 6(x+1) = (x+2)(2x+3).
theorem closed_squares_inner(x: Rat) {
    x * (Rat.2 * x + Rat.1) + Rat.6 * (x + Rat.1) = (x + Rat.2) * (Rat.2 * x + Rat.3)
} by {
    distrib_left(x, Rat.2 * x, Rat.1)
    x * (Rat.2 * x + Rat.1) = x * (Rat.2 * x) + x * Rat.1
    x * (Rat.2 * x) = Rat.2 * (x * x)
    x * Rat.1 = x
    x * (Rat.2 * x + Rat.1) = Rat.2 * (x * x) + x
    distrib_left(Rat.6, x, Rat.1)
    Rat.6 * (x + Rat.1) = Rat.6 * x + Rat.6 * Rat.1
    Rat.6 * Rat.1 = Rat.6
    Rat.6 * (x + Rat.1) = Rat.6 * x + Rat.6
    x * (Rat.2 * x + Rat.1) + Rat.6 * (x + Rat.1) =
        Rat.2 * (x * x) + x + Rat.6 * x + Rat.6
    distrib_left(x + Rat.2, Rat.2 * x, Rat.3)
    (x + Rat.2) * (Rat.2 * x + Rat.3) = (x + Rat.2) * (Rat.2 * x) + (x + Rat.2) * Rat.3
    distrib_right(x, Rat.2, Rat.2 * x)
    (x + Rat.2) * (Rat.2 * x) = x * (Rat.2 * x) + Rat.2 * (Rat.2 * x)
    x * (Rat.2 * x) = Rat.2 * (x * x)
    rat_double_double(x)
    Rat.2 * (Rat.2 * x) = Rat.4 * x
    (x + Rat.2) * (Rat.2 * x) = Rat.2 * (x * x) + Rat.4 * x
    distrib_right(x, Rat.2, Rat.3)
    (x + Rat.2) * Rat.3 = x * Rat.3 + Rat.2 * Rat.3
    rat_two_mul_three
    Rat.2 * Rat.3 = Rat.6
    (x + Rat.2) * Rat.3 = x * Rat.3 + Rat.6
    (x + Rat.2) * (Rat.2 * x + Rat.3) =
        Rat.2 * (x * x) + Rat.4 * x + x * Rat.3 + Rat.6
    x * Rat.3 = Rat.3 * x
    (x + Rat.2) * (Rat.2 * x + Rat.3) =
        Rat.2 * (x * x) + Rat.4 * x + Rat.3 * x + Rat.6
    Rat.1 * x = x
    distrib_right(Rat.1, Rat.6, x)
    (Rat.1 + Rat.6) * x = Rat.1 * x + Rat.6 * x
    rat_one_add_six
    Rat.1 + Rat.6 = Rat.7
    Rat.7 * x = Rat.1 * x + Rat.6 * x
    Rat.1 * x = x
    Rat.7 * x = x + Rat.6 * x
    distrib_right(Rat.4, Rat.3, x)
    (Rat.4 + Rat.3) * x = Rat.4 * x + Rat.3 * x
    rat_four_add_three
    Rat.4 + Rat.3 = Rat.7
    Rat.7 * x = Rat.4 * x + Rat.3 * x
    x + Rat.6 * x = Rat.7 * x
    Rat.4 * x + Rat.3 * x = Rat.7 * x
    x + Rat.6 * x = Rat.4 * x + Rat.3 * x
    Rat.2 * (x * x) + x + Rat.6 * x + Rat.6 =
        Rat.2 * (x * x) + Rat.4 * x + Rat.3 * x + Rat.6
    x * (Rat.2 * x + Rat.1) + Rat.6 * (x + Rat.1) =
        (x + Rat.2) * (Rat.2 * x + Rat.3)
}

/// The polynomial identity behind the closed form of the sum of squares:
/// x(x+1)(2x+1) + 6(x+1)^2 = (x+1)(x+2)(2x+3).
theorem closed_squares_expand(x: Rat) {
    x * (x + Rat.1) * (Rat.2 * x + Rat.1) + Rat.6 * (x + Rat.1) * (x + Rat.1) =
        (x + Rat.1) * (x + Rat.2) * (Rat.2 * x + Rat.3)
} by {
    x * (x + Rat.1) * (Rat.2 * x + Rat.1) =
        (x + Rat.1) * x * (Rat.2 * x + Rat.1)
    Rat.6 * (x + Rat.1) * (x + Rat.1) = (x + Rat.1) * (Rat.6 * (x + Rat.1))
    x * (x + Rat.1) * (Rat.2 * x + Rat.1) + Rat.6 * (x + Rat.1) * (x + Rat.1) =
        (x + Rat.1) * x * (Rat.2 * x + Rat.1) + (x + Rat.1) * (Rat.6 * (x + Rat.1))
    distrib_left(x + Rat.1, x * (Rat.2 * x + Rat.1), Rat.6 * (x + Rat.1))
    (x + Rat.1) * (x * (Rat.2 * x + Rat.1) + Rat.6 * (x + Rat.1)) =
        (x + Rat.1) * (x * (Rat.2 * x + Rat.1)) + (x + Rat.1) * (Rat.6 * (x + Rat.1))
    (x + Rat.1) * x * (Rat.2 * x + Rat.1) + (x + Rat.1) * (Rat.6 * (x + Rat.1)) =
        (x + Rat.1) * (x * (Rat.2 * x + Rat.1) + Rat.6 * (x + Rat.1))
    x * (x + Rat.1) * (Rat.2 * x + Rat.1) + Rat.6 * (x + Rat.1) * (x + Rat.1) =
        (x + Rat.1) * (x * (Rat.2 * x + Rat.1) + Rat.6 * (x + Rat.1))
    closed_squares_inner(x)
    x * (Rat.2 * x + Rat.1) + Rat.6 * (x + Rat.1) = (x + Rat.2) * (Rat.2 * x + Rat.3)
    (x + Rat.1) * (x * (Rat.2 * x + Rat.1) + Rat.6 * (x + Rat.1)) =
        (x + Rat.1) * ((x + Rat.2) * (Rat.2 * x + Rat.3))
    x * (x + Rat.1) * (Rat.2 * x + Rat.1) + Rat.6 * (x + Rat.1) * (x + Rat.1) =
        (x + Rat.1) * ((x + Rat.2) * (Rat.2 * x + Rat.3))
    mul_assoc(x + Rat.1, x + Rat.2, Rat.2 * x + Rat.3)
    (x + Rat.1) * ((x + Rat.2) * (Rat.2 * x + Rat.3)) =
        (x + Rat.1) * (x + Rat.2) * (Rat.2 * x + Rat.3)
    x * (x + Rat.1) * (Rat.2 * x + Rat.1) + Rat.6 * (x + Rat.1) * (x + Rat.1) =
        (x + Rat.1) * (x + Rat.2) * (Rat.2 * x + Rat.3)
}

/// The inductive step of the closed form for the sum of the squares:
/// (x+1)(x+2)(2x+3)/6 = x(x+1)(2x+1)/6 + (x+1)^2 in the rationals.
theorem closed_squares_step(x: Rat) {
    (x + Rat.1) * (x + Rat.2) * (Rat.2 * x + Rat.3) / Rat.6 =
        x * (x + Rat.1) * (Rat.2 * x + Rat.1) / Rat.6 +
            (x + Rat.1) * (x + Rat.1)
} by {
    six_neq_zero
    mul_cancels_div_left((x + Rat.1) * (x + Rat.2) * (Rat.2 * x + Rat.3), Rat.6)
    Rat.6 * ((x + Rat.1) * (x + Rat.2) * (Rat.2 * x + Rat.3) / Rat.6) =
        (x + Rat.1) * (x + Rat.2) * (Rat.2 * x + Rat.3)
    mul_cancels_div(x * (x + Rat.1) * (Rat.2 * x + Rat.1), Rat.6)
    (x * (x + Rat.1) * (Rat.2 * x + Rat.1) / Rat.6) * Rat.6 =
        x * (x + Rat.1) * (Rat.2 * x + Rat.1)
    mul_comm(x * (x + Rat.1) * (Rat.2 * x + Rat.1) / Rat.6, Rat.6)
    Rat.6 * (x * (x + Rat.1) * (Rat.2 * x + Rat.1) / Rat.6) =
        x * (x + Rat.1) * (Rat.2 * x + Rat.1)
    distrib_left(Rat.6, x * (x + Rat.1) * (Rat.2 * x + Rat.1) / Rat.6,
        (x + Rat.1) * (x + Rat.1))
    Rat.6 * (x * (x + Rat.1) * (Rat.2 * x + Rat.1) / Rat.6 + (x + Rat.1) * (x + Rat.1)) =
        Rat.6 * (x * (x + Rat.1) * (Rat.2 * x + Rat.1) / Rat.6) +
            Rat.6 * ((x + Rat.1) * (x + Rat.1))
    Rat.6 * (x * (x + Rat.1) * (Rat.2 * x + Rat.1) / Rat.6 + (x + Rat.1) * (x + Rat.1)) =
        x * (x + Rat.1) * (Rat.2 * x + Rat.1) + Rat.6 * ((x + Rat.1) * (x + Rat.1))
    mul_assoc(Rat.6, x + Rat.1, x + Rat.1)
    Rat.6 * ((x + Rat.1) * (x + Rat.1)) = Rat.6 * (x + Rat.1) * (x + Rat.1)
    Rat.6 * (x * (x + Rat.1) * (Rat.2 * x + Rat.1) / Rat.6 + (x + Rat.1) * (x + Rat.1)) =
        x * (x + Rat.1) * (Rat.2 * x + Rat.1) + Rat.6 * (x + Rat.1) * (x + Rat.1)
    closed_squares_expand(x)
    x * (x + Rat.1) * (Rat.2 * x + Rat.1) + Rat.6 * (x + Rat.1) * (x + Rat.1) =
        (x + Rat.1) * (x + Rat.2) * (Rat.2 * x + Rat.3)
    Rat.6 * (x * (x + Rat.1) * (Rat.2 * x + Rat.1) / Rat.6 + (x + Rat.1) * (x + Rat.1)) =
        (x + Rat.1) * (x + Rat.2) * (Rat.2 * x + Rat.3)
    Rat.6 * ((x + Rat.1) * (x + Rat.2) * (Rat.2 * x + Rat.3) / Rat.6) =
        Rat.6 * (x * (x + Rat.1) * (Rat.2 * x + Rat.1) / Rat.6 + (x + Rat.1) * (x + Rat.1))
    ((x + Rat.1) * (x + Rat.2) * (Rat.2 * x + Rat.3) / Rat.6) * Rat.6 =
        (x * (x + Rat.1) * (Rat.2 * x + Rat.1) / Rat.6 + (x + Rat.1) * (x + Rat.1)) * Rat.6
    mul_cancels_right((x + Rat.1) * (x + Rat.2) * (Rat.2 * x + Rat.3) / Rat.6,
        x * (x + Rat.1) * (Rat.2 * x + Rat.1) / Rat.6 + (x + Rat.1) * (x + Rat.1),
        Rat.6)
    (x + Rat.1) * (x + Rat.2) * (Rat.2 * x + Rat.3) / Rat.6 =
        x * (x + Rat.1) * (Rat.2 * x + Rat.1) / Rat.6 + (x + Rat.1) * (x + Rat.1)
}

/// The closed form for the sum of the squares satisfies its recurrence:
/// closed_squares(k+1) = closed_squares(k) + (k+1)^2.
theorem closed_squares_suc(k: Nat) {
    closed_squares(k.suc) = closed_squares(k) + Rat.from_nat(k.suc.pow(Nat.2))
} by {
    from_nat_suc(k)
    Rat.from_nat(k.suc) = Rat.from_nat(k) + Rat.1
    Rat.from_nat(k.suc) + Rat.1 = Rat.from_nat(k) + Rat.1 + Rat.1
    Rat.1 + Rat.1 = Rat.2
    Rat.from_nat(k.suc) + Rat.1 = Rat.from_nat(k) + Rat.2
    from_nat_mul(Nat.2, k.suc)
    Rat.from_nat(Nat.2) * Rat.from_nat(k.suc) = Rat.from_nat(Nat.2 * k.suc)
    Rat.from_nat(Nat.2) = Rat.2
    Rat.2 * Rat.from_nat(k.suc) = Rat.from_nat(Nat.2 * k.suc)
    Rat.2 * Rat.from_nat(k.suc) = Rat.2 * (Rat.from_nat(k) + Rat.1)
    Rat.from_nat(Nat.2 * k.suc) = Rat.2 * (Rat.from_nat(k) + Rat.1)
    distrib_left(Rat.2, Rat.from_nat(k), Rat.1)
    Rat.2 * (Rat.from_nat(k) + Rat.1) = Rat.2 * Rat.from_nat(k) + Rat.2 * Rat.1
    Rat.2 * Rat.1 = Rat.2
    Rat.2 * (Rat.from_nat(k) + Rat.1) = Rat.2 * Rat.from_nat(k) + Rat.2
    Rat.from_nat(Nat.2 * k.suc) = Rat.2 * Rat.from_nat(k) + Rat.2
    Rat.from_nat(Nat.2 * k.suc) + Rat.1 = Rat.2 * Rat.from_nat(k) + Rat.2 + Rat.1
    Rat.2 + Rat.1 = Rat.3
    Rat.from_nat(Nat.2 * k.suc) + Rat.1 = Rat.2 * Rat.from_nat(k) + Rat.3
    closed_squares(k.suc) =
        Rat.from_nat(k.suc) * (Rat.from_nat(k.suc) + Rat.1) *
            (Rat.from_nat(Nat.2 * k.suc) + Rat.1) / Rat.6
    closed_squares(k.suc) =
        (Rat.from_nat(k) + Rat.1) * (Rat.from_nat(k) + Rat.2) *
            (Rat.2 * Rat.from_nat(k) + Rat.3) / Rat.6
    closed_squares_step(Rat.from_nat(k))
    (Rat.from_nat(k) + Rat.1) * (Rat.from_nat(k) + Rat.2) *
        (Rat.2 * Rat.from_nat(k) + Rat.3) / Rat.6 =
        Rat.from_nat(k) * (Rat.from_nat(k) + Rat.1) *
            (Rat.2 * Rat.from_nat(k) + Rat.1) / Rat.6 +
            (Rat.from_nat(k) + Rat.1) * (Rat.from_nat(k) + Rat.1)
    closed_squares(k) =
        Rat.from_nat(k) * (Rat.from_nat(k) + Rat.1) *
            (Rat.from_nat(Nat.2 * k) + Rat.1) / Rat.6
    from_nat_mul(Nat.2, k)
    Rat.from_nat(Nat.2) * Rat.from_nat(k) = Rat.from_nat(Nat.2 * k)
    Rat.from_nat(Nat.2) = Rat.2
    Rat.2 * Rat.from_nat(k) = Rat.from_nat(Nat.2 * k)
    Rat.from_nat(Nat.2 * k) = Rat.2 * Rat.from_nat(k)
    closed_squares(k) =
        Rat.from_nat(k) * (Rat.from_nat(k) + Rat.1) *
            (Rat.2 * Rat.from_nat(k) + Rat.1) / Rat.6
    closed_squares(k.suc) =
        closed_squares(k) + (Rat.from_nat(k) + Rat.1) * (Rat.from_nat(k) + Rat.1)
    k.suc.pow(Nat.2) = k.suc * k.suc
    Rat.from_nat(k.suc.pow(Nat.2)) = Rat.from_nat(k.suc * k.suc)
    from_nat_mul(k.suc, k.suc)
    Rat.from_nat(k.suc) * Rat.from_nat(k.suc) = Rat.from_nat(k.suc * k.suc)
    Rat.from_nat(k.suc.pow(Nat.2)) =
        Rat.from_nat(k.suc) * Rat.from_nat(k.suc)
    Rat.from_nat(k.suc.pow(Nat.2)) =
        (Rat.from_nat(k) + Rat.1) * (Rat.from_nat(k) + Rat.1)
    closed_squares(k.suc) = closed_squares(k) + Rat.from_nat(k.suc.pow(Nat.2))
}

/// The sum of the first n squares is n(n+1)(2n+1)/6:
/// sum_{k=1}^{n} k^2 = n(n+1)(2n+1)/6.
theorem sum_of_squares(n: Nat) {
    power_sum(Nat.2, n) = closed_squares(n)
} by {
    define p(k: Nat) -> Bool {
        power_sum(Nat.2, k) = closed_squares(k)
    }
    range_sum_one(power_summand(Nat.2))
    range_sum(power_summand(Nat.2), Nat.1) = power_summand(Nat.2, Nat.0)
    power_sum(Nat.2, Nat.0) = range_sum(power_summand(Nat.2), Nat.0.suc)
    Nat.0.suc = Nat.1
    power_sum(Nat.2, Nat.0) = power_summand(Nat.2, Nat.0)
    power_summand(Nat.2, Nat.0) = Rat.from_nat(Nat.0.pow(Nat.2))
    Nat.0.pow(Nat.2) = Nat.0
    Rat.from_nat(Nat.0) = Rat.0
    power_sum(Nat.2, Nat.0) = Rat.0
    closed_squares(Nat.0) =
        Rat.from_nat(Nat.0) * (Rat.from_nat(Nat.0) + Rat.1) *
            (Rat.from_nat(Nat.2 * Nat.0) + Rat.1) / Rat.6
    Rat.from_nat(Nat.0) = Rat.0
    mul_zero_left(Rat.0 + Rat.1)
    Rat.0 * (Rat.0 + Rat.1) = Rat.0
    mul_zero_left(Rat.from_nat(Nat.2 * Nat.0) + Rat.1)
    Rat.0 * (Rat.from_nat(Nat.2 * Nat.0) + Rat.1) = Rat.0
    Rat.0 * (Rat.0 + Rat.1) * (Rat.from_nat(Nat.2 * Nat.0) + Rat.1) = Rat.0
    Rat.0 * (Rat.0 + Rat.1) * (Rat.from_nat(Nat.2 * Nat.0) + Rat.1) / Rat.6 =
        Rat.0 / Rat.6
    Rat.0 / Rat.6 = Rat.0
    Rat.0 * (Rat.0 + Rat.1) * (Rat.from_nat(Nat.2 * Nat.0) + Rat.1) / Rat.6 = Rat.0
    closed_squares(Nat.0) = Rat.0
    power_sum(Nat.2, Nat.0) = closed_squares(Nat.0)
    p(Nat.0)
    forall(k: Nat) {
        if p(k) {
            power_sum(Nat.2, k) = closed_squares(k)
            range_sum_suc(power_summand(Nat.2), k.suc)
            range_sum(power_summand(Nat.2), k.suc.suc) =
                range_sum(power_summand(Nat.2), k.suc) +
                    power_summand(Nat.2, k.suc)
            power_sum(Nat.2, k.suc) =
                range_sum(power_summand(Nat.2), k.suc.suc)
            power_sum(Nat.2, k) = range_sum(power_summand(Nat.2), k.suc)
            power_summand(Nat.2, k.suc) = Rat.from_nat(k.suc.pow(Nat.2))
            power_sum(Nat.2, k.suc) = power_sum(Nat.2, k) + Rat.from_nat(k.suc.pow(Nat.2))
            power_sum(Nat.2, k.suc) = closed_squares(k) + Rat.from_nat(k.suc.pow(Nat.2))
            closed_squares_suc(k)
            closed_squares(k.suc) = closed_squares(k) + Rat.from_nat(k.suc.pow(Nat.2))
            power_sum(Nat.2, k.suc) = closed_squares(k.suc)
            p(k.suc)
        }
    }
    p(Nat.0) and forall(k: Nat) {
        p(k) implies p(k.suc)
    }
    Nat.induction(p)
    p(n)
}

// ---------------------------------------------------------------------------
// The sum of the cubes: 1^3 + ... + n^3 = (n(n+1)/2)^2 = (sum of the first
// naturals)^2.
// ---------------------------------------------------------------------------

/// Two plus two is four, in the rationals.
theorem rat_two_add_two {
    Rat.2 + Rat.2 = Rat.4
} by {
    from_nat_add(Nat.2, Nat.2)
    Rat.from_nat(Nat.2) + Rat.from_nat(Nat.2) = Rat.from_nat(Nat.2 + Nat.2)
    Nat.2 + Nat.2 = Nat.4
    Rat.from_nat(Nat.2) = Rat.2
    Rat.from_nat(Nat.4) = Rat.4
    Rat.2 + Rat.2 = Rat.4
}

/// The inner polynomial identity behind the closed form of the sum of cubes:
/// (x+2)^2 = x^2 + 4(x+1).
theorem closed_cubes_inner(x: Rat) {
    (x + Rat.2) * (x + Rat.2) = x * x + Rat.4 * (x + Rat.1)
} by {
    distrib_left(x + Rat.2, x, Rat.2)
    (x + Rat.2) * (x + Rat.2) = (x + Rat.2) * x + (x + Rat.2) * Rat.2
    distrib_right(x, Rat.2, x)
    (x + Rat.2) * x = x * x + Rat.2 * x
    distrib_right(x, Rat.2, Rat.2)
    (x + Rat.2) * Rat.2 = x * Rat.2 + Rat.2 * Rat.2
    rat_two_mul_two
    Rat.2 * Rat.2 = Rat.4
    (x + Rat.2) * Rat.2 = x * Rat.2 + Rat.4
    (x + Rat.2) * (x + Rat.2) = x * x + Rat.2 * x + x * Rat.2 + Rat.4
    mul_comm(x, Rat.2)
    x * Rat.2 = Rat.2 * x
    (x + Rat.2) * (x + Rat.2) = x * x + Rat.2 * x + Rat.2 * x + Rat.4
    distrib_right(Rat.2, Rat.2, x)
    (Rat.2 + Rat.2) * x = Rat.2 * x + Rat.2 * x
    rat_two_add_two
    Rat.2 + Rat.2 = Rat.4
    Rat.4 * x = Rat.2 * x + Rat.2 * x
    x * x + Rat.2 * x + Rat.2 * x + Rat.4 = x * x + Rat.4 * x + Rat.4
    distrib_left(Rat.4, x, Rat.1)
    Rat.4 * (x + Rat.1) = Rat.4 * x + Rat.4 * Rat.1
    mul_one_right(Rat.4)
    Rat.4 * Rat.1 = Rat.4
    Rat.4 * (x + Rat.1) = Rat.4 * x + Rat.4
    x * x + Rat.4 * (x + Rat.1) = x * x + Rat.4 * x + Rat.4
    (x + Rat.2) * (x + Rat.2) = x * x + Rat.4 * (x + Rat.1)
}

/// The polynomial identity behind the closed form of the sum of cubes:
/// (x+1)^2 (x+2)^2 = x^2 (x+1)^2 + 4 (x+1)^3.
theorem closed_cubes_expand(x: Rat) {
    (x + Rat.1) * (x + Rat.1) * (x + Rat.2) * (x + Rat.2) =
        x * x * (x + Rat.1) * (x + Rat.1) +
            Rat.4 * (x + Rat.1) * (x + Rat.1) * (x + Rat.1)
} by {
    mul_assoc((x + Rat.1) * (x + Rat.1), x + Rat.2, x + Rat.2)
    ((x + Rat.1) * (x + Rat.1) * (x + Rat.2)) * (x + Rat.2) =
        ((x + Rat.1) * (x + Rat.1)) * ((x + Rat.2) * (x + Rat.2))
    (x + Rat.1) * (x + Rat.1) * (x + Rat.2) * (x + Rat.2) =
        ((x + Rat.1) * (x + Rat.1)) * ((x + Rat.2) * (x + Rat.2))
    closed_cubes_inner(x)
    (x + Rat.2) * (x + Rat.2) = x * x + Rat.4 * (x + Rat.1)
    ((x + Rat.1) * (x + Rat.1)) * ((x + Rat.2) * (x + Rat.2)) =
        ((x + Rat.1) * (x + Rat.1)) * (x * x + Rat.4 * (x + Rat.1))
    (x + Rat.1) * (x + Rat.1) * (x + Rat.2) * (x + Rat.2) =
        ((x + Rat.1) * (x + Rat.1)) * (x * x + Rat.4 * (x + Rat.1))
    distrib_left((x + Rat.1) * (x + Rat.1), x * x, Rat.4 * (x + Rat.1))
    ((x + Rat.1) * (x + Rat.1)) * (x * x + Rat.4 * (x + Rat.1)) =
        ((x + Rat.1) * (x + Rat.1)) * (x * x) +
            ((x + Rat.1) * (x + Rat.1)) * (Rat.4 * (x + Rat.1))
    (x + Rat.1) * (x + Rat.1) * (x + Rat.2) * (x + Rat.2) =
        ((x + Rat.1) * (x + Rat.1)) * (x * x) +
            ((x + Rat.1) * (x + Rat.1)) * (Rat.4 * (x + Rat.1))
    ((x + Rat.1) * (x + Rat.1)) * (x * x) =
        x * x * (x + Rat.1) * (x + Rat.1)
    ((x + Rat.1) * (x + Rat.1)) * (Rat.4 * (x + Rat.1)) =
        Rat.4 * (x + Rat.1) * (x + Rat.1) * (x + Rat.1)
    (x + Rat.1) * (x + Rat.1) * (x + Rat.2) * (x + Rat.2) =
        x * x * (x + Rat.1) * (x + Rat.1) +
            Rat.4 * (x + Rat.1) * (x + Rat.1) * (x + Rat.1)
}

/// The inductive step of the closed form for the sum of the cubes:
/// (x+1)^2 (x+2)^2 / 4 = x^2 (x+1)^2 / 4 + (x+1)^3 in the rationals.
theorem closed_cubes_step(x: Rat) {
    (x + Rat.1) * (x + Rat.1) * (x + Rat.2) * (x + Rat.2) / Rat.4 =
        x * x * (x + Rat.1) * (x + Rat.1) / Rat.4 +
            (x + Rat.1) * (x + Rat.1) * (x + Rat.1)
} by {
    four_neq_zero
    mul_cancels_div_left((x + Rat.1) * (x + Rat.1) * (x + Rat.2) * (x + Rat.2), Rat.4)
    Rat.4 * ((x + Rat.1) * (x + Rat.1) * (x + Rat.2) * (x + Rat.2) / Rat.4) =
        (x + Rat.1) * (x + Rat.1) * (x + Rat.2) * (x + Rat.2)
    mul_cancels_div(x * x * (x + Rat.1) * (x + Rat.1), Rat.4)
    (x * x * (x + Rat.1) * (x + Rat.1) / Rat.4) * Rat.4 =
        x * x * (x + Rat.1) * (x + Rat.1)
    mul_comm(x * x * (x + Rat.1) * (x + Rat.1) / Rat.4, Rat.4)
    Rat.4 * (x * x * (x + Rat.1) * (x + Rat.1) / Rat.4) =
        x * x * (x + Rat.1) * (x + Rat.1)
    distrib_left(Rat.4, x * x * (x + Rat.1) * (x + Rat.1) / Rat.4,
        (x + Rat.1) * (x + Rat.1) * (x + Rat.1))
    Rat.4 * (x * x * (x + Rat.1) * (x + Rat.1) / Rat.4 +
        (x + Rat.1) * (x + Rat.1) * (x + Rat.1)) =
        Rat.4 * (x * x * (x + Rat.1) * (x + Rat.1) / Rat.4) +
            Rat.4 * ((x + Rat.1) * (x + Rat.1) * (x + Rat.1))
    Rat.4 * (x * x * (x + Rat.1) * (x + Rat.1) / Rat.4 +
        (x + Rat.1) * (x + Rat.1) * (x + Rat.1)) =
        x * x * (x + Rat.1) * (x + Rat.1) +
            Rat.4 * ((x + Rat.1) * (x + Rat.1) * (x + Rat.1))
    mul_assoc(Rat.4, (x + Rat.1) * (x + Rat.1), x + Rat.1)
    Rat.4 * (((x + Rat.1) * (x + Rat.1)) * (x + Rat.1)) =
        Rat.4 * ((x + Rat.1) * (x + Rat.1)) * (x + Rat.1)
    mul_assoc(Rat.4, x + Rat.1, x + Rat.1)
    Rat.4 * ((x + Rat.1) * (x + Rat.1)) * (x + Rat.1) =
        Rat.4 * (x + Rat.1) * (x + Rat.1) * (x + Rat.1)
    Rat.4 * ((x + Rat.1) * (x + Rat.1) * (x + Rat.1)) =
        Rat.4 * (x + Rat.1) * (x + Rat.1) * (x + Rat.1)
    Rat.4 * (x * x * (x + Rat.1) * (x + Rat.1) / Rat.4 +
        (x + Rat.1) * (x + Rat.1) * (x + Rat.1)) =
        x * x * (x + Rat.1) * (x + Rat.1) +
            Rat.4 * (x + Rat.1) * (x + Rat.1) * (x + Rat.1)
    closed_cubes_expand(x)
    (x + Rat.1) * (x + Rat.1) * (x + Rat.2) * (x + Rat.2) =
        x * x * (x + Rat.1) * (x + Rat.1) +
            Rat.4 * (x + Rat.1) * (x + Rat.1) * (x + Rat.1)
    Rat.4 * (x * x * (x + Rat.1) * (x + Rat.1) / Rat.4 +
        (x + Rat.1) * (x + Rat.1) * (x + Rat.1)) =
        (x + Rat.1) * (x + Rat.1) * (x + Rat.2) * (x + Rat.2)
    Rat.4 * ((x + Rat.1) * (x + Rat.1) * (x + Rat.2) * (x + Rat.2) / Rat.4) =
        Rat.4 * (x * x * (x + Rat.1) * (x + Rat.1) / Rat.4 +
            (x + Rat.1) * (x + Rat.1) * (x + Rat.1))
    ((x + Rat.1) * (x + Rat.1) * (x + Rat.2) * (x + Rat.2) / Rat.4) * Rat.4 =
        (x * x * (x + Rat.1) * (x + Rat.1) / Rat.4 +
            (x + Rat.1) * (x + Rat.1) * (x + Rat.1)) * Rat.4
    mul_cancels_right((x + Rat.1) * (x + Rat.1) * (x + Rat.2) * (x + Rat.2) / Rat.4,
        x * x * (x + Rat.1) * (x + Rat.1) / Rat.4 +
            (x + Rat.1) * (x + Rat.1) * (x + Rat.1),
        Rat.4)
    (x + Rat.1) * (x + Rat.1) * (x + Rat.2) * (x + Rat.2) / Rat.4 =
        x * x * (x + Rat.1) * (x + Rat.1) / Rat.4 +
            (x + Rat.1) * (x + Rat.1) * (x + Rat.1)
}

/// The square of a quotient by two is the quotient of the squares by four:
/// (a/2)^2 = (a*a)/4.
theorem square_div_two(x: Rat) {
    (x / Rat.2).pow(Nat.2) = x * x / Rat.4
} by {
    (x / Rat.2).pow(Nat.2) = (x / Rat.2) * (x / Rat.2)
    mul_fractions(x, Rat.2, x, Rat.2)
    (x / Rat.2) * (x / Rat.2) = (x * x) / (Rat.2 * Rat.2)
    rat_two_mul_two
    Rat.2 * Rat.2 = Rat.4
    (x / Rat.2) * (x / Rat.2) = (x * x) / Rat.4
    (x / Rat.2).pow(Nat.2) = x * x / Rat.4
}

/// A product squared is the product of the squares: (ab)(ab) = aabb.
theorem square_product_rearrange(a: Rat, b: Rat) {
    (a * b) * (a * b) = a * a * b * b
} by {
    mul_assoc(a * b, a, b)
    ((a * b) * a) * b = (a * b) * (a * b)
    (a * b) * a = a * a * b
    ((a * b) * a) * b = (a * a * b) * b
    mul_assoc(a * a, b, b)
    (a * a * b) * b = a * a * (b * b)
    (a * b) * (a * b) = a * a * (b * b)
    (a * b) * (a * b) = a * a * b * b
}

/// The closed form for the sum of the cubes satisfies its recurrence:
/// closed_cubes(k+1) = closed_cubes(k) + (k+1)^3.
theorem closed_cubes_suc(k: Nat) {
    closed_cubes(k.suc) = closed_cubes(k) + Rat.from_nat(k.suc.pow(Nat.3))
} by {
    from_nat_suc(k)
    Rat.from_nat(k.suc) = Rat.from_nat(k) + Rat.1
    Rat.from_nat(k.suc) + Rat.1 = Rat.from_nat(k) + Rat.1 + Rat.1
    Rat.1 + Rat.1 = Rat.2
    Rat.from_nat(k.suc) + Rat.1 = Rat.from_nat(k) + Rat.2
    closed_cubes(k.suc) =
        (Rat.from_nat(k.suc) * (Rat.from_nat(k.suc) + Rat.1) / Rat.2).pow(Nat.2)
    closed_cubes(k.suc) =
        ((Rat.from_nat(k) + Rat.1) * (Rat.from_nat(k) + Rat.2) / Rat.2).pow(Nat.2)
    square_div_two((Rat.from_nat(k) + Rat.1) * (Rat.from_nat(k) + Rat.2))
    ((Rat.from_nat(k) + Rat.1) * (Rat.from_nat(k) + Rat.2) / Rat.2).pow(Nat.2) =
        ((Rat.from_nat(k) + Rat.1) * (Rat.from_nat(k) + Rat.2)) *
            ((Rat.from_nat(k) + Rat.1) * (Rat.from_nat(k) + Rat.2)) / Rat.4
    closed_cubes(k.suc) =
        ((Rat.from_nat(k) + Rat.1) * (Rat.from_nat(k) + Rat.2)) *
            ((Rat.from_nat(k) + Rat.1) * (Rat.from_nat(k) + Rat.2)) / Rat.4
    square_product_rearrange(Rat.from_nat(k) + Rat.1, Rat.from_nat(k) + Rat.2)
    ((Rat.from_nat(k) + Rat.1) * (Rat.from_nat(k) + Rat.2)) *
        ((Rat.from_nat(k) + Rat.1) * (Rat.from_nat(k) + Rat.2)) =
        (Rat.from_nat(k) + Rat.1) * (Rat.from_nat(k) + Rat.1) *
            (Rat.from_nat(k) + Rat.2) * (Rat.from_nat(k) + Rat.2)
    closed_cubes(k.suc) =
        (Rat.from_nat(k) + Rat.1) * (Rat.from_nat(k) + Rat.1) *
            (Rat.from_nat(k) + Rat.2) * (Rat.from_nat(k) + Rat.2) / Rat.4
    closed_cubes(k) =
        (Rat.from_nat(k) * (Rat.from_nat(k) + Rat.1) / Rat.2).pow(Nat.2)
    square_div_two(Rat.from_nat(k) * (Rat.from_nat(k) + Rat.1))
    (Rat.from_nat(k) * (Rat.from_nat(k) + Rat.1) / Rat.2).pow(Nat.2) =
        (Rat.from_nat(k) * (Rat.from_nat(k) + Rat.1)) *
            (Rat.from_nat(k) * (Rat.from_nat(k) + Rat.1)) / Rat.4
    closed_cubes(k) =
        (Rat.from_nat(k) * (Rat.from_nat(k) + Rat.1)) *
            (Rat.from_nat(k) * (Rat.from_nat(k) + Rat.1)) / Rat.4
    square_product_rearrange(Rat.from_nat(k), Rat.from_nat(k) + Rat.1)
    (Rat.from_nat(k) * (Rat.from_nat(k) + Rat.1)) *
        (Rat.from_nat(k) * (Rat.from_nat(k) + Rat.1)) =
        Rat.from_nat(k) * Rat.from_nat(k) *
            (Rat.from_nat(k) + Rat.1) * (Rat.from_nat(k) + Rat.1)
    closed_cubes(k) =
        Rat.from_nat(k) * Rat.from_nat(k) *
            (Rat.from_nat(k) + Rat.1) * (Rat.from_nat(k) + Rat.1) / Rat.4
    closed_cubes_step(Rat.from_nat(k))
    (Rat.from_nat(k) + Rat.1) * (Rat.from_nat(k) + Rat.1) *
        (Rat.from_nat(k) + Rat.2) * (Rat.from_nat(k) + Rat.2) / Rat.4 =
        Rat.from_nat(k) * Rat.from_nat(k) *
            (Rat.from_nat(k) + Rat.1) * (Rat.from_nat(k) + Rat.1) / Rat.4 +
            (Rat.from_nat(k) + Rat.1) * (Rat.from_nat(k) + Rat.1) *
                (Rat.from_nat(k) + Rat.1)
    closed_cubes(k.suc) =
        closed_cubes(k) + (Rat.from_nat(k) + Rat.1) * (Rat.from_nat(k) + Rat.1) *
            (Rat.from_nat(k) + Rat.1)
    k.suc.pow(Nat.3) = k.suc * k.suc * k.suc
    Rat.from_nat(k.suc.pow(Nat.3)) = Rat.from_nat(k.suc * k.suc * k.suc)
    from_nat_mul(k.suc, k.suc * k.suc)
    Rat.from_nat(k.suc) * Rat.from_nat(k.suc * k.suc) =
        Rat.from_nat(k.suc * (k.suc * k.suc))
    Rat.from_nat(k.suc * k.suc * k.suc) = Rat.from_nat(k.suc * (k.suc * k.suc))
    Rat.from_nat(k.suc.pow(Nat.3)) =
        Rat.from_nat(k.suc) * Rat.from_nat(k.suc * k.suc)
    from_nat_mul(k.suc, k.suc)
    Rat.from_nat(k.suc) * Rat.from_nat(k.suc) = Rat.from_nat(k.suc * k.suc)
    Rat.from_nat(k.suc.pow(Nat.3)) =
        Rat.from_nat(k.suc) * (Rat.from_nat(k.suc) * Rat.from_nat(k.suc))
    Rat.from_nat(k.suc.pow(Nat.3)) =
        Rat.from_nat(k.suc) * Rat.from_nat(k.suc) * Rat.from_nat(k.suc)
    Rat.from_nat(k.suc.pow(Nat.3)) =
        (Rat.from_nat(k) + Rat.1) * (Rat.from_nat(k) + Rat.1) *
            (Rat.from_nat(k) + Rat.1)
    closed_cubes(k.suc) = closed_cubes(k) + Rat.from_nat(k.suc.pow(Nat.3))
}

/// The sum of the first n cubes is (n(n+1)/2)^2:
/// sum_{k=1}^{n} k^3 = (n(n+1)/2)^2.
theorem sum_of_cubes(n: Nat) {
    power_sum(Nat.3, n) = closed_cubes(n)
} by {
    define p(k: Nat) -> Bool {
        power_sum(Nat.3, k) = closed_cubes(k)
    }
    range_sum_one(power_summand(Nat.3))
    range_sum(power_summand(Nat.3), Nat.1) = power_summand(Nat.3, Nat.0)
    power_sum(Nat.3, Nat.0) = range_sum(power_summand(Nat.3), Nat.0.suc)
    Nat.0.suc = Nat.1
    power_sum(Nat.3, Nat.0) = power_summand(Nat.3, Nat.0)
    power_summand(Nat.3, Nat.0) = Rat.from_nat(Nat.0.pow(Nat.3))
    Nat.0.pow(Nat.3) = Nat.0
    Rat.from_nat(Nat.0) = Rat.0
    power_sum(Nat.3, Nat.0) = Rat.0
    closed_cubes(Nat.0) =
        (Rat.from_nat(Nat.0) * (Rat.from_nat(Nat.0) + Rat.1) / Rat.2).pow(Nat.2)
    Rat.from_nat(Nat.0) = Rat.0
    mul_zero_left(Rat.0 + Rat.1)
    Rat.0 * (Rat.0 + Rat.1) = Rat.0
    Rat.0 * (Rat.0 + Rat.1) / Rat.2 = Rat.0 / Rat.2
    Rat.0 / Rat.2 = Rat.0
    Rat.0 * (Rat.0 + Rat.1) / Rat.2 = Rat.0
    (Rat.0 * (Rat.0 + Rat.1) / Rat.2).pow(Nat.2) = Rat.0.pow(Nat.2)
    Rat.0.pow(Nat.2) = Rat.0 * Rat.0
    Rat.0 * Rat.0 = Rat.0
    (Rat.0 * (Rat.0 + Rat.1) / Rat.2).pow(Nat.2) = Rat.0
    closed_cubes(Nat.0) = Rat.0
    power_sum(Nat.3, Nat.0) = closed_cubes(Nat.0)
    p(Nat.0)
    forall(k: Nat) {
        if p(k) {
            power_sum(Nat.3, k) = closed_cubes(k)
            range_sum_suc(power_summand(Nat.3), k.suc)
            range_sum(power_summand(Nat.3), k.suc.suc) =
                range_sum(power_summand(Nat.3), k.suc) +
                    power_summand(Nat.3, k.suc)
            power_sum(Nat.3, k.suc) =
                range_sum(power_summand(Nat.3), k.suc.suc)
            power_sum(Nat.3, k) = range_sum(power_summand(Nat.3), k.suc)
            power_summand(Nat.3, k.suc) = Rat.from_nat(k.suc.pow(Nat.3))
            power_sum(Nat.3, k.suc) = power_sum(Nat.3, k) + Rat.from_nat(k.suc.pow(Nat.3))
            power_sum(Nat.3, k.suc) = closed_cubes(k) + Rat.from_nat(k.suc.pow(Nat.3))
            closed_cubes_suc(k)
            closed_cubes(k.suc) = closed_cubes(k) + Rat.from_nat(k.suc.pow(Nat.3))
            power_sum(Nat.3, k.suc) = closed_cubes(k.suc)
            p(k.suc)
        }
    }
    p(Nat.0) and forall(k: Nat) {
        p(k) implies p(k.suc)
    }
    Nat.induction(p)
    p(n)
}

/// The sum of the first n cubes is the square of the sum of the first n
/// naturals: sum_{k=1}^{n} k^3 = (sum_{k=1}^{n} k)^2.
theorem sum_of_cubes_square(n: Nat) {
    power_sum(Nat.3, n) = power_sum(Nat.1, n).pow(Nat.2)
} by {
    sum_of_cubes(n)
    power_sum(Nat.3, n) = closed_cubes(n)
    closed_cubes(n) =
        (Rat.from_nat(n) * (Rat.from_nat(n) + Rat.1) / Rat.2).pow(Nat.2)
    closed_first(n) = Rat.from_nat(n) * (Rat.from_nat(n) + Rat.1) / Rat.2
    closed_cubes(n) = closed_first(n).pow(Nat.2)
    power_sum(Nat.3, n) = closed_first(n).pow(Nat.2)
    sum_of_first(n)
    power_sum(Nat.1, n) = closed_first(n)
    power_sum(Nat.3, n) = power_sum(Nat.1, n).pow(Nat.2)
}

// ---------------------------------------------------------------------------
// The general Faulhaber formula.
// ---------------------------------------------------------------------------

/// The j-th term of the Faulhaber formula for the sum of the p-th powers:
/// (-1)^j * binom(p+1, j) * B_j * n^{p+1-j}.
define faulhaber_term(p: Nat, n: Nat, j: Nat) -> Rat {
    alternating_sign[Rat](j) * Rat.from_nat(p.suc.binom(j)) * bernoulli(j) *
        Rat.from_nat(n).pow(p.suc - j)
}

/// The right-hand side of the Faulhaber formula for the sum of the p-th
/// powers: 1/(p+1) * sum_{j=0}^{p} (-1)^j binom(p+1, j) B_j n^{p+1-j}.
define faulhaber_form(p: Nat, n: Nat) -> Rat {
    Rat.from_nat(p.suc).inverse * range_sum(faulhaber_term(p, n), p.suc)
}

/// The Faulhaber formula at p = 1 specialises to the triangular-number closed
/// form: (1/2)(B_0 n^2 - 2 B_1 n) = n(n+1)/2.
theorem faulhaber_one(n: Nat) {
    faulhaber_form(Nat.1, n) = closed_first(n)
} by {
    range_sum_one(faulhaber_term(Nat.1, n))
    range_sum(faulhaber_term(Nat.1, n), Nat.1) = faulhaber_term(Nat.1, n, Nat.0)
    range_sum_suc(faulhaber_term(Nat.1, n), Nat.1)
    range_sum(faulhaber_term(Nat.1, n), Nat.2) =
        range_sum(faulhaber_term(Nat.1, n), Nat.1) + faulhaber_term(Nat.1, n, Nat.1)
    range_sum(faulhaber_term(Nat.1, n), Nat.2) =
        faulhaber_term(Nat.1, n, Nat.0) + faulhaber_term(Nat.1, n, Nat.1)
    faulhaber_term(Nat.1, n, Nat.0) =
        alternating_sign[Rat](Nat.0) * Rat.from_nat(Nat.2.binom(Nat.0)) *
            bernoulli(Nat.0) * Rat.from_nat(n).pow(Nat.2)
    alternating_sign_zero[Rat]
    alternating_sign[Rat](Nat.0) = Rat.1
    Nat.2.binom(Nat.0) = Nat.1
    Rat.from_nat(Nat.1) = Rat.1
    bernoulli_zero
    bernoulli(Nat.0) = Rat.1
    Rat.from_nat(n).pow(Nat.2) = Rat.from_nat(n) * Rat.from_nat(n)
    faulhaber_term(Nat.1, n, Nat.0) =
        Rat.1 * Rat.1 * Rat.1 * (Rat.from_nat(n) * Rat.from_nat(n))
    Rat.1 * Rat.1 = Rat.1
    Rat.1 * Rat.1 * Rat.1 = Rat.1
    Rat.1 * (Rat.from_nat(n) * Rat.from_nat(n)) = Rat.from_nat(n) * Rat.from_nat(n)
    faulhaber_term(Nat.1, n, Nat.0) = Rat.from_nat(n) * Rat.from_nat(n)
    faulhaber_term(Nat.1, n, Nat.1) =
        alternating_sign[Rat](Nat.1) * Rat.from_nat(Nat.2.binom(Nat.1)) *
            bernoulli(Nat.1) * Rat.from_nat(n).pow(Nat.1)
    alternating_sign_suc[Rat](Nat.0)
    alternating_sign[Rat](Nat.1) = -alternating_sign[Rat](Nat.0)
    alternating_sign_zero[Rat]
    alternating_sign[Rat](Nat.1) = -Rat.1
    Nat.2.binom(Nat.1) = Nat.2
    Rat.from_nat(Nat.2) = Rat.2
    bernoulli_one
    bernoulli(Nat.1) = -Rat.1 / Rat.2
    Rat.from_nat(n).pow(Nat.1) = Rat.from_nat(n)
    faulhaber_term(Nat.1, n, Nat.1) =
        (-Rat.1) * Rat.2 * (-Rat.1 / Rat.2) * Rat.from_nat(n)
    mul_neg_left(Rat.1, Rat.2)
    (-Rat.1) * Rat.2 = -(Rat.1 * Rat.2)
    Rat.1 * Rat.2 = Rat.2
    (-Rat.1) * Rat.2 = -Rat.2
    (-Rat.1) * Rat.2 * (-Rat.1 / Rat.2) = (-Rat.2) * (-Rat.1 / Rat.2)
    mul_neg_neg(Rat.2, Rat.1 / Rat.2)
    (-Rat.2) * (-Rat.1 / Rat.2) = Rat.2 * (Rat.1 / Rat.2)
    mul_cancels_div(Rat.1, Rat.2)
    (Rat.1 / Rat.2) * Rat.2 = Rat.1
    mul_comm(Rat.1 / Rat.2, Rat.2)
    Rat.2 * (Rat.1 / Rat.2) = Rat.1
    (-Rat.2) * (-Rat.1 / Rat.2) = Rat.1
    (-Rat.1) * Rat.2 * (-Rat.1 / Rat.2) = Rat.1
    faulhaber_term(Nat.1, n, Nat.1) = Rat.1 * Rat.from_nat(n)
    mul_one_left(Rat.from_nat(n))
    Rat.1 * Rat.from_nat(n) = Rat.from_nat(n)
    faulhaber_term(Nat.1, n, Nat.1) = Rat.from_nat(n)
    range_sum(faulhaber_term(Nat.1, n), Nat.2) =
        Rat.from_nat(n) * Rat.from_nat(n) + Rat.from_nat(n)
    faulhaber_form(Nat.1, n) =
        Rat.from_nat(Nat.2).inverse * range_sum(faulhaber_term(Nat.1, n), Nat.2)
    Rat.from_nat(Nat.2) = Rat.2
    faulhaber_form(Nat.1, n) =
        Rat.2.inverse * (Rat.from_nat(n) * Rat.from_nat(n) + Rat.from_nat(n))
    distrib_left(Rat.from_nat(n), Rat.from_nat(n), Rat.1)
    Rat.from_nat(n) * (Rat.from_nat(n) + Rat.1) =
        Rat.from_nat(n) * Rat.from_nat(n) + Rat.from_nat(n) * Rat.1
    mul_one_right(Rat.from_nat(n))
    Rat.from_nat(n) * Rat.1 = Rat.from_nat(n)
    Rat.from_nat(n) * (Rat.from_nat(n) + Rat.1) =
        Rat.from_nat(n) * Rat.from_nat(n) + Rat.from_nat(n)
    faulhaber_form(Nat.1, n) =
        Rat.2.inverse * (Rat.from_nat(n) * (Rat.from_nat(n) + Rat.1))
    mul_comm(Rat.2.inverse, Rat.from_nat(n) * (Rat.from_nat(n) + Rat.1))
    Rat.2.inverse * (Rat.from_nat(n) * (Rat.from_nat(n) + Rat.1)) =
        (Rat.from_nat(n) * (Rat.from_nat(n) + Rat.1)) * Rat.2.inverse
    faulhaber_form(Nat.1, n) =
        (Rat.from_nat(n) * (Rat.from_nat(n) + Rat.1)) * Rat.2.inverse
    closed_first(n) = Rat.from_nat(n) * (Rat.from_nat(n) + Rat.1) / Rat.2
    faulhaber_form(Nat.1, n) = closed_first(n)
}

// The general Faulhaber formula.
//
//   sum_{k=1}^{n} k^p = 1/(p+1) sum_{j=0}^{p} (-1)^j binom(p+1, j) B_j n^{p+1-j}
//
// In the library's notation, with `power_sum` as above and the Bernoulli
// numbers B_j from number_theory/bernoulli.ac (B_1 = -1/2), this reads
//
//   power_sum(p, n) = faulhaber_form(p, n)
//
// The three small cases above are exactly the instances p = 1, 2, 3 of this
// formula (and `faulhaber_one` checks the p = 1 instance directly against the
// definition of `faulhaber_form`).  A proof of the general statement would
// expand (k - 1)^{p+1} by the binomial theorem, sum the result over k using
// the identity sum_{k=1}^{n} binom(k, j) = binom(n+1, j+1), and use the
// Bernoulli recurrence; the library has the binomial theorem for rings
// (comm_ring/binomial.ac) and the Bernoulli recurrence
// (bernoulli_recurrence in number_theory/bernoulli.ac), but the full
// formalisation is left for future work.
//
// theorem faulhaber_formula(p: Nat, n: Nat) {
//     p >= Nat.1 implies power_sum(p, n) = faulhaber_form(p, n)
// }
