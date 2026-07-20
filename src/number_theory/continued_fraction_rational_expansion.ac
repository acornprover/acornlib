from nat import Nat, add_suc_left, add_suc_right, add_zero_left,
    add_zero_right, div_mod_decomp, lt_and_lte, lt_diff, lt_not_ref, lt_suc,
    lt_suc_right, lt_trans, pos_of_ne_zero, strong_induction, true_below,
    true_below_apply, zero_or_suc
from order import lt_imp_lte
from rat import Rat, from_nat_add, from_nat_mul, mul_div_cancels,
    nat_lt_imp_rat_lt
from list import List
from number_theory.continued_fraction import ContinuedFraction,
    continued_fraction_new_self,
    continued_fraction_value_eq_coefficients_value,
    finite_continued_fraction_coefficients,
    continued_fraction_value, continued_fraction_value_cons,
    continued_fraction_value_singleton,
    positive_continued_fraction_tail,
    positive_continued_fraction_tail_nil,
    positive_continued_fraction_tail_cons_intro
from number_theory.continued_fraction_value_ratio import rat_add_div,
    rat_div_inverse_swap

numerals Nat

/// The Euclidean quotient-remainder expansion of a natural fraction within a
/// finite number of steps.
define rational_continued_fraction_coefficients_fuel(
    numerator: Nat, denominator: Nat, fuel: Nat
) -> List[Nat] {
    match fuel {
        Nat.zero {
            List.nil[Nat]
        }
        Nat.suc(remaining) {
            if denominator = Nat.0 {
                List.nil[Nat]
            } else {
                let quotient = numerator.div(denominator)
                let remainder = numerator.mod(denominator)
                if remainder = Nat.0 {
                    List.cons(quotient, List.nil[Nat])
                } else {
                    List.cons(quotient,
                        rational_continued_fraction_coefficients_fuel(
                            denominator, remainder, remaining))
                }
            }
        }
    }
}

/// The Euclidean continued-fraction coefficients of a natural fraction.
define rational_continued_fraction_coefficients(
    numerator: Nat, denominator: Nat
) -> List[Nat] {
    rational_continued_fraction_coefficients_fuel(
        numerator, denominator, denominator.suc)
}

/// Zero fuel gives no Euclidean coefficients.
theorem rational_continued_fraction_coefficients_fuel_zero(
    numerator: Nat, denominator: Nat
) {
    rational_continued_fraction_coefficients_fuel(
        numerator, denominator, Nat.0) = List.nil[Nat]
}

/// A zero denominator gives no Euclidean coefficients.
theorem rational_continued_fraction_coefficients_fuel_suc_denominator_zero(
    numerator: Nat, remaining: Nat
) {
    rational_continued_fraction_coefficients_fuel(
        numerator, Nat.0, remaining.suc) = List.nil[Nat]
}

/// A Euclidean step with zero remainder consists only of its quotient.
theorem rational_continued_fraction_coefficients_fuel_suc_remainder_zero(
    numerator: Nat, denominator: Nat, remaining: Nat
) {
    denominator != Nat.0 and numerator.mod(denominator) = Nat.0 implies
        rational_continued_fraction_coefficients_fuel(
            numerator, denominator, remaining.suc) =
            List.cons(numerator.div(denominator), List.nil[Nat])
}

/// A Euclidean step with nonzero remainder continues with the denominator and
/// remainder interchanged.
theorem rational_continued_fraction_coefficients_fuel_suc_remainder_nonzero(
    numerator: Nat, denominator: Nat, remaining: Nat
) {
    denominator != Nat.0 and numerator.mod(denominator) != Nat.0 implies
        rational_continued_fraction_coefficients_fuel(
            numerator, denominator, remaining.suc) =
            List.cons(numerator.div(denominator),
                rational_continued_fraction_coefficients_fuel(
                    denominator, numerator.mod(denominator), remaining))
}

/// The coefficient expansion with zero denominator is empty.
theorem rational_continued_fraction_coefficients_denominator_zero(
    numerator: Nat
) {
    rational_continued_fraction_coefficients(numerator, Nat.0) =
        List.nil[Nat]
} by {
    rational_continued_fraction_coefficients_fuel_suc_denominator_zero(
        numerator, Nat.0)
}

/// An exact natural quotient expands to its singleton coefficient.
theorem rational_continued_fraction_coefficients_remainder_zero(
    numerator: Nat, denominator: Nat
) {
    denominator != Nat.0 and numerator.mod(denominator) = Nat.0 implies
        rational_continued_fraction_coefficients(numerator, denominator) =
            List.cons(numerator.div(denominator), List.nil[Nat])
} by {
    if denominator != Nat.0 and numerator.mod(denominator) = Nat.0 {
        rational_continued_fraction_coefficients_fuel_suc_remainder_zero(
            numerator, denominator, denominator)
    }
}

/// A nonterminal natural fraction begins with its quotient and continues from
/// its denominator and remainder.
theorem rational_continued_fraction_coefficients_remainder_nonzero(
    numerator: Nat, denominator: Nat
) {
    denominator != Nat.0 and numerator.mod(denominator) != Nat.0 implies
        rational_continued_fraction_coefficients(numerator, denominator) =
            List.cons(numerator.div(denominator),
                rational_continued_fraction_coefficients_fuel(
                    denominator, numerator.mod(denominator), denominator))
} by {
    if denominator != Nat.0 and numerator.mod(denominator) != Nat.0 {
        rational_continued_fraction_coefficients_fuel_suc_remainder_nonzero(
            numerator, denominator, denominator)
    }
}

/// The remainder of division by a nonzero denominator is smaller than that
/// denominator.
theorem rational_continued_fraction_remainder_lt(
    numerator: Nat, denominator: Nat
) {
    denominator != Nat.0 implies
        numerator.mod(denominator) < denominator
}

/// Division by a nonzero denominator no larger than the numerator has positive
/// quotient.
theorem rational_continued_fraction_quotient_nonzero(
    numerator: Nat, denominator: Nat
) {
    denominator != Nat.0 and denominator <= numerator implies
        numerator.div(denominator) != Nat.0
} by {
    if denominator != Nat.0 and denominator <= numerator {
        if numerator.div(denominator) = Nat.0 {
            div_mod_decomp(numerator, denominator)
            numerator.div(denominator) * denominator = Nat.0
            numerator.div(denominator) * denominator +
                numerator.mod(denominator) = numerator
            Nat.0 + numerator.mod(denominator) = numerator
            add_zero_left(numerator.mod(denominator))
            numerator.mod(denominator) = numerator
            rational_continued_fraction_remainder_lt(numerator, denominator)
            numerator < denominator
            lt_and_lte(numerator, denominator, numerator)
            numerator < numerator
            lt_not_ref(numerator)
            false
        }
    }
}

/// Division by a nonzero denominator no larger than the numerator has positive
/// quotient.
theorem rational_continued_fraction_quotient_positive(
    numerator: Nat, denominator: Nat
) {
    denominator != Nat.0 and denominator <= numerator implies
        Nat.0 < numerator.div(denominator)
} by {
    if denominator != Nat.0 and denominator <= numerator {
        rational_continued_fraction_quotient_nonzero(numerator, denominator)
        pos_of_ne_zero(numerator.div(denominator))
        Nat.0 < numerator.div(denominator)
    }
}

/// Once the fuel exceeds the denominator, one more step does not change the
/// Euclidean coefficient expansion.
theorem rational_continued_fraction_coefficients_fuel_stable_step(
    remaining: Nat
) {
    (forall(a: Nat, b: Nat) {
        b < remaining implies
            rational_continued_fraction_coefficients_fuel(a, b, remaining) =
            rational_continued_fraction_coefficients_fuel(
                a, b, remaining.suc)
    }) implies forall(a: Nat, b: Nat) {
        b < remaining.suc implies
            rational_continued_fraction_coefficients_fuel(
                a, b, remaining.suc) =
            rational_continued_fraction_coefficients_fuel(
                a, b, remaining.suc.suc)
    }
} by {
    if forall(a: Nat, b: Nat) {
        b < remaining implies
            rational_continued_fraction_coefficients_fuel(a, b, remaining) =
            rational_continued_fraction_coefficients_fuel(
                a, b, remaining.suc)
    } {
        forall(a: Nat, b: Nat) {
            if b < remaining.suc {
                if b = Nat.0 {
                    rational_continued_fraction_coefficients_fuel_suc_denominator_zero(
                        a, remaining)
                    rational_continued_fraction_coefficients_fuel_suc_denominator_zero(
                        a, remaining.suc)
                    rational_continued_fraction_coefficients_fuel(
                        a, b, remaining.suc) =
                        rational_continued_fraction_coefficients_fuel(
                            a, b, remaining.suc.suc)
                } else {
                    if a.mod(b) = Nat.0 {
                        rational_continued_fraction_coefficients_fuel_suc_remainder_zero(
                            a, b, remaining)
                        rational_continued_fraction_coefficients_fuel_suc_remainder_zero(
                            a, b, remaining.suc)
                        rational_continued_fraction_coefficients_fuel(
                            a, b, remaining.suc) =
                            rational_continued_fraction_coefficients_fuel(
                                a, b, remaining.suc.suc)
                    } else {
                        rational_continued_fraction_remainder_lt(a, b)
                        lt_suc_right(b, remaining)
                        if b = remaining {
                            a.mod(b) < remaining
                        } else {
                            b < remaining
                            lt_trans(a.mod(b), b, remaining)
                        }
                        rational_continued_fraction_coefficients_fuel(
                            b, a.mod(b), remaining) =
                            rational_continued_fraction_coefficients_fuel(
                                b, a.mod(b), remaining.suc)
                        rational_continued_fraction_coefficients_fuel_suc_remainder_nonzero(
                            a, b, remaining)
                        rational_continued_fraction_coefficients_fuel_suc_remainder_nonzero(
                            a, b, remaining.suc)
                        rational_continued_fraction_coefficients_fuel(
                            a, b, remaining.suc) =
                            rational_continued_fraction_coefficients_fuel(
                                a, b, remaining.suc.suc)
                    }
                }
                rational_continued_fraction_coefficients_fuel(
                    a, b, remaining.suc) =
                    rational_continued_fraction_coefficients_fuel(
                        a, b, remaining.suc.suc)
            }
        }
    }
}

/// Once the fuel exceeds the denominator, one more step does not change the
/// Euclidean coefficient expansion.
theorem rational_continued_fraction_coefficients_fuel_stable(
    numerator: Nat, denominator: Nat, fuel: Nat
) {
    denominator < fuel implies
        rational_continued_fraction_coefficients_fuel(
            numerator, denominator, fuel) =
        rational_continued_fraction_coefficients_fuel(
            numerator, denominator, fuel.suc)
} by {
    define p(steps: Nat) -> Bool {
        forall(a: Nat, b: Nat) {
            b < steps implies
                rational_continued_fraction_coefficients_fuel(a, b, steps) =
                rational_continued_fraction_coefficients_fuel(
                    a, b, steps.suc)
        }
    }
    p(Nat.0)
    forall(remaining: Nat) {
        if p(remaining) {
            rational_continued_fraction_coefficients_fuel_stable_step(
                remaining)
            p(remaining.suc)
        }
    }
    p(Nat.0) and forall(remaining: Nat) {
        p(remaining) implies p(remaining.suc)
    }
    Nat.induction(p)
    p(fuel)
}

/// Once the fuel exceeds the denominator, any additional fuel gives the same
/// Euclidean coefficient expansion.
theorem rational_continued_fraction_coefficients_fuel_stable_additional_step(
    numerator: Nat, denominator: Nat, fuel: Nat, extra: Nat
) {
    denominator < fuel and
        rational_continued_fraction_coefficients_fuel(
            numerator, denominator, fuel) =
        rational_continued_fraction_coefficients_fuel(
            numerator, denominator, fuel + extra) implies
        rational_continued_fraction_coefficients_fuel(
            numerator, denominator, fuel) =
        rational_continued_fraction_coefficients_fuel(
            numerator, denominator, fuel + extra.suc)
} by {
    if denominator < fuel and
            rational_continued_fraction_coefficients_fuel(
                numerator, denominator, fuel) =
            rational_continued_fraction_coefficients_fuel(
                numerator, denominator, fuel + extra) {
        fuel <= fuel + extra
        lt_and_lte(denominator, fuel, fuel + extra)
        rational_continued_fraction_coefficients_fuel_stable(
            numerator, denominator, fuel + extra)
        add_suc_right(fuel, extra)
        rational_continued_fraction_coefficients_fuel(
            numerator, denominator, fuel) =
            rational_continued_fraction_coefficients_fuel(
                numerator, denominator, fuel + extra.suc)
    }
}

/// Once the fuel exceeds the denominator, any additional fuel gives the same
/// Euclidean coefficient expansion.
theorem rational_continued_fraction_coefficients_fuel_stable_additional(
    numerator: Nat, denominator: Nat, fuel: Nat, additional: Nat
) {
    denominator < fuel implies
        rational_continued_fraction_coefficients_fuel(
            numerator, denominator, fuel) =
        rational_continued_fraction_coefficients_fuel(
            numerator, denominator, fuel + additional)
} by {
    define p(extra: Nat) -> Bool {
        rational_continued_fraction_coefficients_fuel(
            numerator, denominator, fuel) =
        rational_continued_fraction_coefficients_fuel(
            numerator, denominator, fuel + extra)
    }
    if denominator < fuel {
        add_zero_right(fuel)
        p(Nat.0)
        forall(extra: Nat) {
            if p(extra) {
                rational_continued_fraction_coefficients_fuel_stable_additional_step(
                    numerator, denominator, fuel, extra)
                p(extra.suc)
            }
        }
        p(Nat.0) and forall(extra: Nat) {
            p(extra) implies p(extra.suc)
        }
        Nat.induction(p)
        p(additional)
    }
}

/// One more than the denominator is sufficient fuel for the Euclidean
/// coefficient expansion.
theorem rational_continued_fraction_coefficients_fuel_sufficient(
    numerator: Nat, denominator: Nat, additional: Nat
) {
    rational_continued_fraction_coefficients_fuel(
        numerator, denominator, denominator.suc + additional) =
        rational_continued_fraction_coefficients(numerator, denominator)
} by {
    lt_suc(denominator)
    rational_continued_fraction_coefficients_fuel_stable_additional(
        numerator, denominator, denominator.suc, additional)
}

/// Fuel larger than the denominator gives the complete Euclidean coefficient
/// expansion.
theorem rational_continued_fraction_coefficients_fuel_complete(
    numerator: Nat, denominator: Nat, fuel: Nat
) {
    denominator < fuel implies
        rational_continued_fraction_coefficients_fuel(
            numerator, denominator, fuel) =
        rational_continued_fraction_coefficients(numerator, denominator)
} by {
    if denominator < fuel {
        lt_diff(denominator, fuel)
        let gap: Nat satisfy {
            denominator + gap = fuel and gap != Nat.0
        }
        zero_or_suc(gap)
        let additional: Nat satisfy { gap = additional.suc }
        add_suc_right(denominator, additional)
        add_suc_left(denominator, additional)
        denominator.suc + additional = fuel
        rational_continued_fraction_coefficients_fuel_sufficient(
            numerator, denominator, additional)
        rational_continued_fraction_coefficients_fuel(
            numerator, denominator, fuel) =
            rational_continued_fraction_coefficients(numerator, denominator)
    }
}

/// A nonterminal expansion continues with the complete expansion of the
/// denominator by the preceding remainder.
theorem rational_continued_fraction_coefficients_remainder_nonzero_swapped(
    numerator: Nat, denominator: Nat
) {
    denominator != Nat.0 and numerator.mod(denominator) != Nat.0 implies
        rational_continued_fraction_coefficients(numerator, denominator) =
            List.cons(
                numerator.div(denominator),
                rational_continued_fraction_coefficients(
                    denominator, numerator.mod(denominator)))
} by {
    if denominator != Nat.0 and numerator.mod(denominator) != Nat.0 {
        rational_continued_fraction_remainder_lt(numerator, denominator)
        rational_continued_fraction_coefficients_fuel_complete(
            denominator, numerator.mod(denominator), denominator)
        rational_continued_fraction_coefficients_fuel(
            denominator, numerator.mod(denominator), denominator) =
            rational_continued_fraction_coefficients(
                denominator, numerator.mod(denominator))
        rational_continued_fraction_coefficients_remainder_nonzero(
            numerator, denominator)
        rational_continued_fraction_coefficients(numerator, denominator) =
            List.cons(
                numerator.div(denominator),
                rational_continued_fraction_coefficients_fuel(
                    denominator, numerator.mod(denominator), denominator))
        List.cons(
            numerator.div(denominator),
            rational_continued_fraction_coefficients_fuel(
                denominator, numerator.mod(denominator), denominator)) =
            List.cons(
                numerator.div(denominator),
                rational_continued_fraction_coefficients(
                    denominator, numerator.mod(denominator)))
        rational_continued_fraction_coefficients(numerator, denominator) =
            List.cons(
                numerator.div(denominator),
                rational_continued_fraction_coefficients(
                    denominator, numerator.mod(denominator)))
    }
}

/// The tail of a nonterminal expansion is the complete expansion of the
/// denominator by the preceding remainder.
theorem rational_continued_fraction_coefficients_tail_swapped(
    numerator: Nat, denominator: Nat, head: Nat, tail: List[Nat]
) {
    denominator != Nat.0 and numerator.mod(denominator) != Nat.0 and
        rational_continued_fraction_coefficients(numerator, denominator) =
            List.cons(head, tail) implies
        tail = rational_continued_fraction_coefficients(
            denominator, numerator.mod(denominator))
} by {
    if denominator != Nat.0 and numerator.mod(denominator) != Nat.0 and
            rational_continued_fraction_coefficients(
                numerator, denominator) = List.cons(head, tail) {
        rational_continued_fraction_coefficients_remainder_nonzero_swapped(
            numerator, denominator)
        List.cons(head, tail) =
            List.cons(
                numerator.div(denominator),
                rational_continued_fraction_coefficients(
                    denominator, numerator.mod(denominator)))
        tail = rational_continued_fraction_coefficients(
            denominator, numerator.mod(denominator))
    }
}

/// Reconstruction of a continued-fraction value across one Euclidean
/// quotient-remainder step.
theorem rational_continued_fraction_value_reconstruction_step(
    numerator: Nat, denominator: Nat, tail: List[Nat]
) {
    denominator != Nat.0 and numerator.mod(denominator) != Nat.0 and
        continued_fraction_value(tail) =
            Rat.from_nat(denominator) /
                Rat.from_nat(numerator.mod(denominator)) implies
        continued_fraction_value(List.cons(
            numerator.div(denominator), tail)) =
            Rat.from_nat(numerator) / Rat.from_nat(denominator)
} by {
    if denominator != Nat.0 and numerator.mod(denominator) != Nat.0 and
            continued_fraction_value(tail) =
                Rat.from_nat(denominator) /
                    Rat.from_nat(numerator.mod(denominator)) {
        pos_of_ne_zero(denominator)
        nat_lt_imp_rat_lt(Nat.0, denominator)
        Rat.from_nat(Nat.0) = Rat.0
        Rat.from_nat(denominator) != Rat.0
        continued_fraction_value_cons(numerator.div(denominator), tail)
        rat_div_inverse_swap(
            Rat.from_nat(denominator),
            Rat.from_nat(numerator.mod(denominator)))
        continued_fraction_value(tail).inverse =
            Rat.from_nat(numerator.mod(denominator)) /
                Rat.from_nat(denominator)
        rat_add_div(
            Rat.from_nat(numerator.div(denominator)),
            Rat.from_nat(numerator.mod(denominator)),
            Rat.from_nat(denominator))
        from_nat_mul(numerator.div(denominator), denominator)
        from_nat_add(
            numerator.div(denominator) * denominator,
            numerator.mod(denominator))
        div_mod_decomp(numerator, denominator)
        Rat.from_nat(numerator.div(denominator)) *
                Rat.from_nat(denominator) +
                Rat.from_nat(numerator.mod(denominator)) =
            Rat.from_nat(numerator)
        continued_fraction_value(List.cons(
            numerator.div(denominator), tail)) =
            Rat.from_nat(numerator) / Rat.from_nat(denominator)
    }
}

/// Reconstruction of a Euclidean coefficient expansion from reconstruction of
/// its possible nonterminal tail.
theorem rational_continued_fraction_coefficients_value_induction_step(
    numerator: Nat, denominator: Nat
) {
    denominator != Nat.0 and
        (numerator.mod(denominator) != Nat.0 implies
            continued_fraction_value(
                rational_continued_fraction_coefficients(
                    denominator, numerator.mod(denominator))) =
            Rat.from_nat(denominator) /
                Rat.from_nat(numerator.mod(denominator))) implies
        continued_fraction_value(
            rational_continued_fraction_coefficients(numerator, denominator)) =
        Rat.from_nat(numerator) / Rat.from_nat(denominator)
} by {
    if denominator != Nat.0 {
        pos_of_ne_zero(denominator)
        nat_lt_imp_rat_lt(Nat.0, denominator)
        Rat.from_nat(Nat.0) = Rat.0
        Rat.from_nat(denominator) != Rat.0
        if numerator.mod(denominator) = Nat.0 {
            rational_continued_fraction_coefficients_remainder_zero(
                numerator, denominator)
            continued_fraction_value_singleton(
                numerator.div(denominator))
            div_mod_decomp(numerator, denominator)
            add_zero_right(numerator.div(denominator) * denominator)
            numerator.div(denominator) * denominator = numerator
            from_nat_mul(numerator.div(denominator), denominator)
            mul_div_cancels(
                Rat.from_nat(numerator.div(denominator)),
                Rat.from_nat(denominator))
            Rat.from_nat(numerator) / Rat.from_nat(denominator) =
                Rat.from_nat(numerator.div(denominator))
            continued_fraction_value(
                rational_continued_fraction_coefficients(
                    numerator, denominator)) =
                Rat.from_nat(numerator) / Rat.from_nat(denominator)
        } else {
            rational_continued_fraction_value_reconstruction_step(
                numerator,
                denominator,
                rational_continued_fraction_coefficients(
                    denominator, numerator.mod(denominator)))
            rational_continued_fraction_coefficients_remainder_nonzero_swapped(
                numerator, denominator)
            continued_fraction_value(
                rational_continued_fraction_coefficients(
                    numerator, denominator)) =
                Rat.from_nat(numerator) / Rat.from_nat(denominator)
        }
    }
}

/// The Euclidean coefficient expansion reconstructs its original natural
/// numerator and positive denominator as a rational value.
theorem rational_continued_fraction_coefficients_value(
    numerator: Nat, denominator: Nat
) {
    denominator != Nat.0 implies
        continued_fraction_value(
            rational_continued_fraction_coefficients(numerator, denominator)) =
        Rat.from_nat(numerator) / Rat.from_nat(denominator)
} by {
    let p: Nat -> Bool = function(current_denominator: Nat) {
        forall(current_numerator: Nat) {
            current_denominator != Nat.0 implies
                continued_fraction_value(
                    rational_continued_fraction_coefficients(
                        current_numerator, current_denominator)) =
                Rat.from_nat(current_numerator) /
                    Rat.from_nat(current_denominator)
        }
    }
    forall(current_denominator: Nat) {
        if true_below(p, current_denominator) {
            forall(current_numerator: Nat) {
                if current_denominator != Nat.0 {
                    if current_numerator.mod(current_denominator) != Nat.0 {
                        rational_continued_fraction_remainder_lt(
                            current_numerator, current_denominator)
                        true_below_apply(
                            p,
                            current_denominator,
                            current_numerator.mod(current_denominator))
                        p(current_numerator.mod(current_denominator))
                        continued_fraction_value(
                            rational_continued_fraction_coefficients(
                                current_denominator,
                                current_numerator.mod(current_denominator))) =
                            Rat.from_nat(current_denominator) /
                                Rat.from_nat(
                                    current_numerator.mod(current_denominator))
                    }
                    rational_continued_fraction_coefficients_value_induction_step(
                        current_numerator, current_denominator)
                    continued_fraction_value(
                        rational_continued_fraction_coefficients(
                            current_numerator, current_denominator)) =
                        Rat.from_nat(current_numerator) /
                            Rat.from_nat(current_denominator)
                }
            }
            p(current_denominator)
        }
    }
    forall(current_denominator: Nat) {
        true_below(p, current_denominator) implies p(current_denominator)
    }
    strong_induction(p)
    forall(current_denominator: Nat) { p(current_denominator) }
    if denominator != Nat.0 {
        continued_fraction_value(
            rational_continued_fraction_coefficients(numerator, denominator)) =
        Rat.from_nat(numerator) / Rat.from_nat(denominator)
    }
}

/// Every coefficient produced from a fraction at least one is positive.
theorem rational_continued_fraction_coefficients_fuel_positive(
    numerator: Nat, denominator: Nat, fuel: Nat
) {
    denominator != Nat.0 and denominator <= numerator implies
        positive_continued_fraction_tail(
            rational_continued_fraction_coefficients_fuel(
                numerator, denominator, fuel))
} by {
    define p(steps: Nat) -> Bool {
        forall(a: Nat, b: Nat) {
            b != Nat.0 and b <= a implies
                positive_continued_fraction_tail(
                    rational_continued_fraction_coefficients_fuel(
                        a, b, steps))
        }
    }
    forall(a: Nat, b: Nat) {
        rational_continued_fraction_coefficients_fuel_zero(a, b)
        positive_continued_fraction_tail_nil
        positive_continued_fraction_tail(
            rational_continued_fraction_coefficients_fuel(a, b, Nat.0))
    }
    p(Nat.0)
    forall(remaining: Nat) {
        if p(remaining) {
            forall(a: Nat, b: Nat) {
                if b != Nat.0 and b <= a {
                    rational_continued_fraction_quotient_positive(a, b)
                    if a.mod(b) = Nat.0 {
                        rational_continued_fraction_coefficients_fuel_suc_remainder_zero(
                            a, b, remaining)
                        rational_continued_fraction_coefficients_fuel(
                            a, b, remaining.suc) =
                            List.cons(a.div(b), List.nil[Nat])
                        positive_continued_fraction_tail_nil
                        positive_continued_fraction_tail_cons_intro(
                            a.div(b), List.nil[Nat])
                        positive_continued_fraction_tail(
                            rational_continued_fraction_coefficients_fuel(
                                a, b, remaining.suc))
                    } else {
                        rational_continued_fraction_remainder_lt(a, b)
                        lt_imp_lte(a.mod(b), b)
                        a.mod(b) <= b
                        p(remaining)
                        positive_continued_fraction_tail(
                            rational_continued_fraction_coefficients_fuel(
                                b, a.mod(b), remaining))
                        rational_continued_fraction_coefficients_fuel_suc_remainder_nonzero(
                            a, b, remaining)
                        rational_continued_fraction_coefficients_fuel(
                            a, b, remaining.suc) =
                            List.cons(
                                a.div(b),
                                rational_continued_fraction_coefficients_fuel(
                                    b, a.mod(b), remaining))
                        positive_continued_fraction_tail_cons_intro(
                            a.div(b),
                            rational_continued_fraction_coefficients_fuel(
                                b, a.mod(b), remaining))
                        positive_continued_fraction_tail(
                            rational_continued_fraction_coefficients_fuel(
                                a, b, remaining.suc))
                    }
                }
            }
            p(remaining.suc)
        }
    }
    p(Nat.0) and forall(remaining: Nat) {
        p(remaining) implies p(remaining.suc)
    }
    Nat.induction(p)
    p(fuel)
}

/// A positive denominator gives a head coefficient and a positive coefficient
/// tail.
theorem rational_continued_fraction_coefficients_head_positive_tail(
    numerator: Nat, denominator: Nat
) {
    denominator != Nat.0 implies
        exists(head: Nat, tail: List[Nat]) {
            rational_continued_fraction_coefficients(numerator, denominator) =
                List.cons(head, tail) and
            positive_continued_fraction_tail(tail)
        }
} by {
    if denominator != Nat.0 {
        if numerator.mod(denominator) = Nat.0 {
            rational_continued_fraction_coefficients_remainder_zero(
                numerator, denominator)
            positive_continued_fraction_tail_nil
            exists(head: Nat, tail: List[Nat]) {
                rational_continued_fraction_coefficients(
                    numerator, denominator) = List.cons(head, tail) and
                positive_continued_fraction_tail(tail)
            }
        } else {
            rational_continued_fraction_remainder_lt(numerator, denominator)
            lt_imp_lte(numerator.mod(denominator), denominator)
            rational_continued_fraction_coefficients_fuel_positive(
                denominator, numerator.mod(denominator), denominator)
            rational_continued_fraction_coefficients_remainder_nonzero(
                numerator, denominator)
            exists(head: Nat, tail: List[Nat]) {
                rational_continued_fraction_coefficients(
                    numerator, denominator) = List.cons(head, tail) and
                positive_continued_fraction_tail(tail)
            }
        }
    }
}

/// A positive denominator gives a nonempty Euclidean coefficient list.
theorem rational_continued_fraction_coefficients_nonempty(
    numerator: Nat, denominator: Nat
) {
    denominator != Nat.0 implies
        rational_continued_fraction_coefficients(numerator, denominator) !=
            List.nil[Nat]
} by {
    if denominator != Nat.0 {
        if numerator.mod(denominator) = Nat.0 {
            rational_continued_fraction_coefficients_remainder_zero(
                numerator, denominator)
            List.cons(numerator.div(denominator), List.nil[Nat]) !=
                List.nil[Nat]
            rational_continued_fraction_coefficients(
                numerator, denominator) != List.nil[Nat]
        } else {
            rational_continued_fraction_coefficients_remainder_nonzero(
                numerator, denominator)
            List.cons(
                numerator.div(denominator),
                rational_continued_fraction_coefficients_fuel(
                    denominator, numerator.mod(denominator), denominator)) !=
                List.nil[Nat]
            rational_continued_fraction_coefficients(
                numerator, denominator) != List.nil[Nat]
        }
    }
}

/// All Euclidean coefficients after the first are positive.
theorem rational_continued_fraction_coefficients_positive_tail(
    numerator: Nat, denominator: Nat, head: Nat, tail: List[Nat]
) {
    denominator != Nat.0 and
        rational_continued_fraction_coefficients(numerator, denominator) =
            List.cons(head, tail) implies
        positive_continued_fraction_tail(tail)
} by {
    if denominator != Nat.0 and
            rational_continued_fraction_coefficients(numerator, denominator) =
                List.cons(head, tail) {
        rational_continued_fraction_coefficients_head_positive_tail(
            numerator, denominator)
        let found_head: Nat satisfy {
            exists(found_tail: List[Nat]) {
                rational_continued_fraction_coefficients(
                    numerator, denominator) =
                    List.cons(found_head, found_tail) and
                positive_continued_fraction_tail(found_tail)
            }
        }
        let found_tail: List[Nat] satisfy {
            rational_continued_fraction_coefficients(numerator, denominator) =
                List.cons(found_head, found_tail) and
            positive_continued_fraction_tail(found_tail)
        }
        List.cons(head, tail) = List.cons(found_head, found_tail)
        tail = found_tail
        positive_continued_fraction_tail(tail)
    }
}

/// Positive denominators yield valid finite continued-fraction coefficients.
theorem rational_continued_fraction_coefficients_valid(
    numerator: Nat, denominator: Nat
) {
    denominator != Nat.0 implies
        finite_continued_fraction_coefficients(
            rational_continued_fraction_coefficients(numerator, denominator))
} by {
    if denominator != Nat.0 {
        rational_continued_fraction_coefficients_head_positive_tail(
            numerator, denominator)
        let head: Nat satisfy {
            exists(tail: List[Nat]) {
                rational_continued_fraction_coefficients(
                    numerator, denominator) = List.cons(head, tail) and
                positive_continued_fraction_tail(tail)
            }
        }
        let tail: List[Nat] satisfy {
            rational_continued_fraction_coefficients(numerator, denominator) =
                List.cons(head, tail) and
            positive_continued_fraction_tail(tail)
        }
        rational_continued_fraction_coefficients(numerator, denominator) =
            List.cons(head, tail)
        positive_continued_fraction_tail(tail)
        finite_continued_fraction_coefficients(
            rational_continued_fraction_coefficients(
                numerator, denominator))
    }
}

/// The finite continued fraction obtained from the Euclidean coefficient
/// expansion when those coefficients are valid.
define rational_continued_fraction(
    numerator: Nat, denominator: Nat
) -> Option[ContinuedFraction] {
    ContinuedFraction.new(
        rational_continued_fraction_coefficients(numerator, denominator))
}

/// A positive denominator produces a finite continued fraction.
theorem rational_continued_fraction_some(
    numerator: Nat, denominator: Nat
) {
    denominator != Nat.0 implies
        exists(cf: ContinuedFraction) {
            rational_continued_fraction(numerator, denominator) =
                Option.some(cf)
        }
} by {
    if denominator != Nat.0 {
        rational_continued_fraction_coefficients_valid(
            numerator, denominator)
        exists(cf: ContinuedFraction) {
            ContinuedFraction.new(
                rational_continued_fraction_coefficients(
                    numerator, denominator)) = Option.some(cf)
        }
        exists(cf: ContinuedFraction) {
            rational_continued_fraction(numerator, denominator) =
                Option.some(cf)
        }
    }
}

/// A constructed rational continued fraction has the Euclidean coefficient
/// list.
theorem rational_continued_fraction_some_coefficients(
    numerator: Nat, denominator: Nat, cf: ContinuedFraction
) {
    rational_continued_fraction(numerator, denominator) = Option.some(cf)
        implies cf.coefficients =
            rational_continued_fraction_coefficients(numerator, denominator)
} by {
    if rational_continued_fraction(numerator, denominator) = Option.some(cf) {
        ContinuedFraction.new(
            rational_continued_fraction_coefficients(
                numerator, denominator)) = Option.some(cf)
        continued_fraction_new_self(cf)
        ContinuedFraction.new(cf.coefficients) = Option.some(cf)
        cf.coefficients =
            rational_continued_fraction_coefficients(numerator, denominator)
    }
}

/// A constructed rational continued fraction has the value of its original
/// natural numerator and positive denominator.
theorem rational_continued_fraction_some_value(
    numerator: Nat, denominator: Nat, cf: ContinuedFraction
) {
    denominator != Nat.0 and
        rational_continued_fraction(numerator, denominator) = Option.some(cf)
        implies cf.value =
            Rat.from_nat(numerator) / Rat.from_nat(denominator)
} by {
    if denominator != Nat.0 and
            rational_continued_fraction(numerator, denominator) =
                Option.some(cf) {
        rational_continued_fraction_some_coefficients(
            numerator, denominator, cf)
        cf.coefficients =
            rational_continued_fraction_coefficients(numerator, denominator)
        continued_fraction_value_eq_coefficients_value(cf)
        rational_continued_fraction_coefficients_value(
            numerator, denominator)
        cf.value = Rat.from_nat(numerator) / Rat.from_nat(denominator)
    }
}
