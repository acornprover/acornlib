from nat import Nat, add_assoc, add_comm, add_comm_4, distrib_left, distrib_right,
    from_nat_one, from_nat_zero, mul_assoc, mul_comm, mul_one_left, mul_one_right,
    mul_zero_left, mul_zero_right
from rat import Rat, add_div_distrib, from_nat_add, from_nat_mul, inverse_inverts,
    mul_div_cancels, nat_lt_imp_rat_lt, pos_ne_zero, recip_mul, recip_recip
from list import List
from number_theory.continued_fraction import continuant, continuant_state,
    continuant_state_nil, continuant_state_cons, continuant_nil, continuant_singleton,
    continuant_positive_of_positive_tail, continued_fraction_value,
    continued_fraction_value_cons, continued_fraction_value_singleton,
    continued_fraction_numerator, continued_fraction_denominator,
    continued_fraction_convergent, ContinuedFraction,
    continued_fraction_coefficients_valid,
    continued_fraction_value_eq_coefficients_value,
    continued_fraction_numerator_eq_coefficients_numerator,
    continued_fraction_denominator_eq_coefficients_denominator,
    continued_fraction_convergent_eq_coefficients_convergent,
    continued_fraction_numerator_eq_continuant_of_valid_coefficients,
    continued_fraction_denominator_cons,
    continued_fraction_convergent_first_eq_numerator,
    continued_fraction_convergent_second_eq_denominator,
    continued_fraction_prefix_coefficients, continued_fraction_prefix_value,
    continued_fraction_prefix_numerator, continued_fraction_prefix_denominator,
    continued_fraction_prefix_convergent,
    finite_continued_fraction_coefficients,
    finite_continued_fraction_coefficients_cons,
    finite_continued_fraction_prefix_coefficients,
    finite_continued_fraction_prefix_coefficients_elim,
    continued_fraction_has_valid_prefix_elim,
    continued_fraction_prefix_value_eq_coefficients_prefix_value,
    continued_fraction_prefix_numerator_eq_coefficients_prefix_numerator,
    continued_fraction_prefix_denominator_eq_coefficients_prefix_denominator,
    continued_fraction_prefix_convergent_eq_coefficients_prefix_convergent,
    positive_continued_fraction_tail, positive_continued_fraction_tail_cons_tail

numerals Nat

/// The reciprocal of a rational quotient interchanges its numerator and
/// denominator.
theorem rat_div_inverse_swap(numerator: Rat, denominator: Rat) {
    (numerator / denominator).inverse = denominator / numerator
} by {
    recip_mul(numerator, denominator.inverse)
    recip_recip(denominator)
    numerator.inverse * denominator = denominator * numerator.inverse
}

/// Adding a rational quotient to a whole rational combines over its nonzero
/// denominator.
theorem rat_add_div(
    whole: Rat, numerator: Rat, denominator: Rat
) {
    denominator != Rat.0 implies
        whole + numerator / denominator =
            (whole * denominator + numerator) / denominator
} by {
    if denominator != Rat.0 {
        add_div_distrib(whole * denominator, numerator, denominator)
        mul_div_cancels(whole, denominator)
        (whole * denominator + numerator) / denominator =
            (whole * denominator) / denominator + numerator / denominator
        (whole * denominator) / denominator = whole
        (whole * denominator + numerator) / denominator =
            whole + numerator / denominator
        whole + numerator / denominator =
            (whole * denominator + numerator) / denominator
    }
}

/// A continuant state depends linearly on its two initial values.
theorem continuant_state_linear(
    coefficients: List[Nat], previous: Nat, current: Nat
) {
    continuant_state(coefficients, previous, current) =
        previous * continuant_state(coefficients, Nat.1, Nat.0) +
        current * continuant_state(coefficients, Nat.0, Nat.1)
} by {
    define p(items: List[Nat]) -> Bool {
        forall(previous1: Nat, current1: Nat) {
            continuant_state(items, previous1, current1) =
                previous1 * continuant_state(items, Nat.1, Nat.0) +
                current1 * continuant_state(items, Nat.0, Nat.1)
        }
    }
    continuant_state_nil(Nat.1, Nat.0)
    continuant_state_nil(Nat.0, Nat.1)
    forall(previous1: Nat, current1: Nat) {
        continuant_state_nil(previous1, current1)
        mul_zero_right(previous1)
        mul_one_right(current1)
        previous1 * Nat.0 + current1 * Nat.1 = Nat.0 + current1
        previous1 * Nat.0 + current1 * Nat.1 = current1
        continuant_state(List.nil[Nat], previous1, current1) = current1
        previous1 * continuant_state(List.nil[Nat], Nat.1, Nat.0) +
            current1 * continuant_state(List.nil[Nat], Nat.0, Nat.1) = current1
        continuant_state(List.nil[Nat], previous1, current1) =
            previous1 * continuant_state(List.nil[Nat], Nat.1, Nat.0) +
            current1 * continuant_state(List.nil[Nat], Nat.0, Nat.1)
    }
    p(List.nil[Nat])
    forall(head: Nat, tail: List[Nat]) {
        if p(tail) {
            forall(previous1: Nat, current1: Nat) {
                continuant_state_cons(head, tail, previous1, current1)
                continuant_state_cons(head, tail, Nat.1, Nat.0)
                continuant_state_cons(head, tail, Nat.0, Nat.1)
                continuant_state(tail, current1, current1 * head + previous1) =
                    current1 * continuant_state(tail, Nat.1, Nat.0) +
                    (current1 * head + previous1) *
                        continuant_state(tail, Nat.0, Nat.1)
                continuant_state(tail, Nat.1, head) =
                    Nat.1 * continuant_state(tail, Nat.1, Nat.0) +
                    head * continuant_state(tail, Nat.0, Nat.1)
                Nat.1 * continuant_state(tail, Nat.1, Nat.0) =
                    continuant_state(tail, Nat.1, Nat.0)
                continuant_state(tail, Nat.1, head) =
                    continuant_state(tail, Nat.1, Nat.0) +
                    head * continuant_state(tail, Nat.0, Nat.1)
                continuant_state(tail, Nat.0, Nat.1) =
                    Nat.0 * continuant_state(tail, Nat.1, Nat.0) +
                    Nat.1 * continuant_state(tail, Nat.0, Nat.1)
                let a = continuant_state(tail, Nat.1, Nat.0)
                let b = continuant_state(tail, Nat.0, Nat.1)
                current1 * (a + head * b) =
                    current1 * a + current1 * (head * b)
                current1 * (head * b) = (current1 * head) * b
                (current1 * head + previous1) * b =
                    (current1 * head) * b + previous1 * b
                previous1 * b + (current1 * a + (current1 * head) * b) =
                    current1 * a + ((current1 * head) * b + previous1 * b)
                previous1 * continuant_state(tail, Nat.0, Nat.1) +
                    current1 * continuant_state(tail, Nat.1, head) =
                    current1 * continuant_state(tail, Nat.1, Nat.0) +
                    (current1 * head + previous1) *
                        continuant_state(tail, Nat.0, Nat.1)
                continuant_state(List.cons(head, tail), previous1, current1) =
                    previous1 * continuant_state(
                        List.cons(head, tail), Nat.1, Nat.0) +
                    current1 * continuant_state(
                        List.cons(head, tail), Nat.0, Nat.1)
            }
            p(List.cons(head, tail))
        }
    }
    forall(head: Nat, tail: List[Nat]) {
        p(tail) implies p(List.cons(head, tail))
    }
    p(List.nil[Nat]) and forall(head: Nat, tail: List[Nat]) {
        p(tail) implies p(List.cons(head, tail))
    }
    List.induction(p)
    forall(items1: List[Nat]) { p(items1) }
    p(coefficients)
}

/// A continuant with at least two coefficients satisfies the head recurrence.
theorem continuant_cons_cons(
    head: Nat, next: Nat, tail: List[Nat]
) {
    continuant(List.cons(head, List.cons(next, tail))) =
        head * continuant(List.cons(next, tail)) + continuant(tail)
} by {
    continuant_state_cons(head, List.cons(next, tail), Nat.0, Nat.1)
    continuant_state_linear(List.cons(next, tail), Nat.1, head)
    continuant_state_cons(next, tail, Nat.1, Nat.0)
    continuant_state_cons(next, tail, Nat.0, Nat.1)
    mul_zero_left(next)
    mul_one_left(next)
    continuant_state(tail, Nat.0, Nat.1) = continuant(tail)
    continuant_state(List.cons(next, tail), Nat.0, Nat.1) =
        continuant(List.cons(next, tail))
    Nat.1 * continuant_state(List.cons(next, tail), Nat.1, Nat.0) =
        continuant_state(List.cons(next, tail), Nat.1, Nat.0)
    continuant_state(List.cons(next, tail), Nat.1, Nat.0) = continuant(tail)
    continuant_state(List.cons(next, tail), Nat.1, head) =
        continuant(tail) + head * continuant(List.cons(next, tail))
    continuant(tail) + head * continuant(List.cons(next, tail)) =
        head * continuant(List.cons(next, tail)) + continuant(tail)
}

/// The value of a finite simple continued fraction is the quotient of the
/// continuant of its coefficients by the continuant of its tail.
theorem continued_fraction_value_eq_continuant_ratio(
    head: Nat, tail: List[Nat]
) {
    finite_continued_fraction_coefficients(List.cons(head, tail)) implies
        continued_fraction_value(List.cons(head, tail)) =
            Rat.from_nat(continuant(List.cons(head, tail))) /
                Rat.from_nat(continuant(tail))
} by {
    define p(items: List[Nat]) -> Bool {
        forall(first: Nat, rest: List[Nat]) {
            items = List.cons(first, rest)
                and finite_continued_fraction_coefficients(items)
                implies continued_fraction_value(items) =
                    Rat.from_nat(continuant(items)) /
                        Rat.from_nat(continuant(rest))
        }
    }
    forall(first: Nat, rest: List[Nat]) {
        not List.nil[Nat] = List.cons(first, rest)
    }
    p(List.nil[Nat])
    forall(list_head: Nat, list_tail: List[Nat]) {
        if p(list_tail) {
            forall(first: Nat, rest: List[Nat]) {
                if List.cons(list_head, list_tail) = List.cons(first, rest)
                    and finite_continued_fraction_coefficients(
                        List.cons(list_head, list_tail)) {
                    list_head = first
                    list_tail = rest
                    match list_tail {
                    List.nil {
                        continued_fraction_value_singleton(list_head)
                        continuant_nil
                        continuant_singleton(list_head)
                        from_nat_one[Rat]
                        Rat.from_nat(continuant(List.nil[Nat])) = Rat.1
                        Rat.from_nat(continuant(List.cons(list_head, List.nil[Nat]))) =
                            Rat.from_nat(list_head)
                        Rat.from_nat(list_head) / Rat.1 = Rat.from_nat(list_head)
                        continued_fraction_value(
                            List.cons(list_head, List.nil[Nat])) =
                            Rat.from_nat(continuant(
                                List.cons(list_head, List.nil[Nat]))) /
                                Rat.from_nat(continuant(List.nil[Nat]))
                        continued_fraction_value(List.cons(list_head, list_tail)) =
                            Rat.from_nat(continuant(
                                List.cons(list_head, list_tail))) /
                                Rat.from_nat(continuant(list_tail))
                    }
                    List.cons(second, remaining) {
                        finite_continued_fraction_coefficients_cons(list_head, list_tail)
                        positive_continued_fraction_tail(list_tail)
                        positive_continued_fraction_tail_cons_tail(second, remaining)
                        positive_continued_fraction_tail(remaining)
                        finite_continued_fraction_coefficients_cons(second, remaining)
                        finite_continued_fraction_coefficients(list_tail)
                        continued_fraction_value(list_tail) =
                            Rat.from_nat(continuant(list_tail)) /
                                Rat.from_nat(continuant(remaining))
                        rat_div_inverse_swap(
                            Rat.from_nat(continuant(list_tail)),
                            Rat.from_nat(continuant(remaining)))
                        continued_fraction_value_cons(list_head, list_tail)
                        continuant_positive_of_positive_tail(list_tail)
                        Nat.0 < continuant(list_tail)
                        nat_lt_imp_rat_lt(Nat.0, continuant(list_tail))
                        Rat.from_nat(Nat.0) < Rat.from_nat(continuant(list_tail))
                        from_nat_zero[Rat]
                        Rat.from_nat(Nat.0) = Rat.0
                        Rat.0 < Rat.from_nat(continuant(list_tail))
                        Rat.from_nat(continuant(list_tail)).is_positive
                        pos_ne_zero(Rat.from_nat(continuant(list_tail)))
                        rat_add_div(
                            Rat.from_nat(list_head),
                            Rat.from_nat(continuant(remaining)),
                            Rat.from_nat(continuant(list_tail)))
                        from_nat_mul(list_head, continuant(list_tail))
                        from_nat_add(
                            list_head * continuant(list_tail), continuant(remaining))
                        continuant_cons_cons(list_head, second, remaining)
                        continued_fraction_value(List.cons(list_head, list_tail)) =
                            Rat.from_nat(continuant(
                                List.cons(list_head, list_tail))) /
                                Rat.from_nat(continuant(list_tail))
                    }
                    }
                    continuant(rest) = continuant(list_tail)
                    Rat.from_nat(continuant(rest)) =
                        Rat.from_nat(continuant(list_tail))
                    Rat.from_nat(continuant(List.cons(list_head, list_tail))) /
                        Rat.from_nat(continuant(rest)) =
                        Rat.from_nat(continuant(List.cons(list_head, list_tail))) /
                            Rat.from_nat(continuant(list_tail))
                    continued_fraction_value(List.cons(list_head, list_tail)) =
                        Rat.from_nat(continuant(List.cons(list_head, list_tail))) /
                            Rat.from_nat(continuant(list_tail))
                    Rat.from_nat(continuant(List.cons(list_head, list_tail))) /
                        Rat.from_nat(continuant(list_tail)) =
                        Rat.from_nat(continuant(List.cons(list_head, list_tail))) /
                            Rat.from_nat(continuant(rest))
                    continued_fraction_value(List.cons(list_head, list_tail)) =
                        Rat.from_nat(continuant(List.cons(list_head, list_tail))) /
                            Rat.from_nat(continuant(rest))
                }
            }
            p(List.cons(list_head, list_tail))
        }
    }
    forall(list_head: Nat, list_tail: List[Nat]) {
        p(list_tail) implies p(List.cons(list_head, list_tail))
    }
    p(List.nil[Nat]) and forall(list_head: Nat, list_tail: List[Nat]) {
        p(list_tail) implies p(List.cons(list_head, list_tail))
    }
    List.induction(p)
    forall(items: List[Nat]) { p(items) }
    p(List.cons(head, tail))
    List.cons(head, tail) = List.cons(head, tail)
}

/// The value of a valid coefficient list is its numerator divided by its
/// denominator.
theorem continued_fraction_value_eq_numerator_div_denominator(
    coefficients: List[Nat]
) {
    finite_continued_fraction_coefficients(coefficients) implies
        continued_fraction_value(coefficients) =
            Rat.from_nat(continued_fraction_numerator(coefficients)) /
                Rat.from_nat(continued_fraction_denominator(coefficients))
} by {
    if finite_continued_fraction_coefficients(coefficients) {
        match coefficients {
            List.nil {
                not finite_continued_fraction_coefficients(List.nil[Nat])
                continued_fraction_value(coefficients) =
                    Rat.from_nat(continued_fraction_numerator(coefficients)) /
                        Rat.from_nat(continued_fraction_denominator(coefficients))
            }
            List.cons(head, tail) {
                coefficients = List.cons(head, tail)
                continued_fraction_value_eq_continuant_ratio(head, tail)
                continued_fraction_value(List.cons(head, tail)) =
                    Rat.from_nat(continuant(List.cons(head, tail))) /
                        Rat.from_nat(continuant(tail))
                continued_fraction_value(coefficients) =
                    continued_fraction_value(List.cons(head, tail))
                continuant(coefficients) = continuant(List.cons(head, tail))
                Rat.from_nat(continuant(coefficients)) =
                    Rat.from_nat(continuant(List.cons(head, tail)))
                Rat.from_nat(continuant(coefficients)) /
                    Rat.from_nat(continuant(tail)) =
                    Rat.from_nat(continuant(List.cons(head, tail))) /
                        Rat.from_nat(continuant(tail))
                continued_fraction_value(coefficients) =
                    Rat.from_nat(continuant(coefficients)) /
                        Rat.from_nat(continuant(tail))
                continued_fraction_numerator_eq_continuant_of_valid_coefficients(
                    coefficients)
                continued_fraction_denominator_cons(head, tail)
                continued_fraction_numerator(coefficients) = continuant(coefficients)
                continued_fraction_denominator(coefficients) = continuant(tail)
                Rat.from_nat(continued_fraction_numerator(coefficients)) =
                    Rat.from_nat(continuant(coefficients))
                Rat.from_nat(continued_fraction_denominator(coefficients)) =
                    Rat.from_nat(continuant(tail))
                Rat.from_nat(continued_fraction_numerator(coefficients)) /
                    Rat.from_nat(continued_fraction_denominator(coefficients)) =
                    Rat.from_nat(continuant(coefficients)) /
                        Rat.from_nat(continuant(tail))
                continued_fraction_value(coefficients) =
                    Rat.from_nat(continued_fraction_numerator(coefficients)) /
                        Rat.from_nat(continued_fraction_denominator(coefficients))
            }
        }
        continued_fraction_value(coefficients) =
            Rat.from_nat(continued_fraction_numerator(coefficients)) /
                Rat.from_nat(continued_fraction_denominator(coefficients))
    }
}

/// The value of a valid coefficient list is the quotient of the two
/// projections of its convergent pair.
theorem continued_fraction_value_eq_convergent_first_div_second(
    coefficients: List[Nat]
) {
    finite_continued_fraction_coefficients(coefficients) implies
        continued_fraction_value(coefficients) =
            Rat.from_nat(continued_fraction_convergent(coefficients).first) /
                Rat.from_nat(continued_fraction_convergent(coefficients).second)
} by {
    if finite_continued_fraction_coefficients(coefficients) {
        continued_fraction_value_eq_numerator_div_denominator(coefficients)
        continued_fraction_convergent_first_eq_numerator(coefficients)
        continued_fraction_convergent_second_eq_denominator(coefficients)
        Rat.from_nat(continued_fraction_convergent(coefficients).first) =
            Rat.from_nat(continued_fraction_numerator(coefficients))
        Rat.from_nat(continued_fraction_convergent(coefficients).second) =
            Rat.from_nat(continued_fraction_denominator(coefficients))
        Rat.from_nat(continued_fraction_convergent(coefficients).first) /
            Rat.from_nat(continued_fraction_convergent(coefficients).second) =
            Rat.from_nat(continued_fraction_numerator(coefficients)) /
                Rat.from_nat(continued_fraction_denominator(coefficients))
        continued_fraction_value(coefficients) =
            Rat.from_nat(continued_fraction_numerator(coefficients)) /
                Rat.from_nat(continued_fraction_denominator(coefficients))
    }
}

/// The value of a finite simple continued fraction is its numerator divided by
/// its denominator.
theorem continued_fraction_value_eq_numerator_div_denominator_method(
    cf: ContinuedFraction
) {
    cf.value = Rat.from_nat(cf.numerator) / Rat.from_nat(cf.denominator)
} by {
    continued_fraction_coefficients_valid(cf)
    continued_fraction_value_eq_numerator_div_denominator(cf.coefficients)
    continued_fraction_value_eq_coefficients_value(cf)
    continued_fraction_numerator_eq_coefficients_numerator(cf)
    continued_fraction_denominator_eq_coefficients_denominator(cf)
}

/// The value of a finite simple continued fraction is the quotient of its
/// convergent projections.
theorem continued_fraction_value_eq_convergent_first_div_second_method(
    cf: ContinuedFraction
) {
    cf.value = Rat.from_nat(cf.convergent.first) /
        Rat.from_nat(cf.convergent.second)
} by {
    continued_fraction_coefficients_valid(cf)
    continued_fraction_value_eq_convergent_first_div_second(cf.coefficients)
    continued_fraction_value_eq_coefficients_value(cf)
    continued_fraction_convergent_eq_coefficients_convergent(cf)
}

/// The value of a valid prefix is its prefix numerator divided by its prefix
/// denominator.
theorem continued_fraction_prefix_value_eq_numerator_div_denominator(
    coefficients: List[Nat], n: Nat
) {
    finite_continued_fraction_prefix_coefficients(coefficients, n) implies
        continued_fraction_prefix_value(coefficients, n) =
            Rat.from_nat(continued_fraction_prefix_numerator(coefficients, n)) /
                Rat.from_nat(continued_fraction_prefix_denominator(coefficients, n))
} by {
    if finite_continued_fraction_prefix_coefficients(coefficients, n) {
        finite_continued_fraction_prefix_coefficients_elim(coefficients, n)
        continued_fraction_value_eq_numerator_div_denominator(
            continued_fraction_prefix_coefficients(coefficients, n))
        continued_fraction_prefix_value(coefficients, n) =
            continued_fraction_value(
                continued_fraction_prefix_coefficients(coefficients, n))
        continued_fraction_prefix_numerator(coefficients, n) =
            continued_fraction_numerator(
                continued_fraction_prefix_coefficients(coefficients, n))
        continued_fraction_prefix_denominator(coefficients, n) =
            continued_fraction_denominator(
                continued_fraction_prefix_coefficients(coefficients, n))
        continued_fraction_prefix_value(coefficients, n) =
            Rat.from_nat(continued_fraction_prefix_numerator(coefficients, n)) /
                Rat.from_nat(continued_fraction_prefix_denominator(coefficients, n))
    }
}

/// The value of a valid prefix is the quotient of the two projections of its
/// prefix convergent.
theorem continued_fraction_prefix_value_eq_convergent_first_div_second(
    coefficients: List[Nat], n: Nat
) {
    finite_continued_fraction_prefix_coefficients(coefficients, n) implies
        continued_fraction_prefix_value(coefficients, n) =
            Rat.from_nat(continued_fraction_prefix_convergent(
                coefficients, n).first) /
                Rat.from_nat(continued_fraction_prefix_convergent(
                    coefficients, n).second)
} by {
    if finite_continued_fraction_prefix_coefficients(coefficients, n) {
        finite_continued_fraction_prefix_coefficients_elim(coefficients, n)
        continued_fraction_value_eq_convergent_first_div_second(
            continued_fraction_prefix_coefficients(coefficients, n))
        continued_fraction_prefix_value(coefficients, n) =
            continued_fraction_value(
                continued_fraction_prefix_coefficients(coefficients, n))
        continued_fraction_prefix_convergent(coefficients, n) =
            continued_fraction_convergent(
                continued_fraction_prefix_coefficients(coefficients, n))
        continued_fraction_prefix_value(coefficients, n) =
            Rat.from_nat(continued_fraction_prefix_convergent(
                coefficients, n).first) /
                Rat.from_nat(continued_fraction_prefix_convergent(
                    coefficients, n).second)
    }
}

/// The value of a valid continued-fraction prefix is its prefix numerator
/// divided by its prefix denominator.
theorem continued_fraction_prefix_value_eq_numerator_div_denominator_method(
    cf: ContinuedFraction, n: Nat
) {
    cf.has_valid_prefix(n) implies
        cf.prefix_value(n) = Rat.from_nat(cf.prefix_numerator(n)) /
            Rat.from_nat(cf.prefix_denominator(n))
} by {
    if cf.has_valid_prefix(n) {
        continued_fraction_has_valid_prefix_elim(cf, n)
        continued_fraction_prefix_value_eq_numerator_div_denominator(
            cf.coefficients, n)
        continued_fraction_prefix_value_eq_coefficients_prefix_value(cf, n)
        continued_fraction_prefix_numerator_eq_coefficients_prefix_numerator(cf, n)
        continued_fraction_prefix_denominator_eq_coefficients_prefix_denominator(cf, n)
        cf.prefix_value(n) = Rat.from_nat(cf.prefix_numerator(n)) /
            Rat.from_nat(cf.prefix_denominator(n))
    }
}

/// The value of a valid continued-fraction prefix is the quotient of its
/// prefix-convergent projections.
theorem continued_fraction_prefix_value_eq_convergent_first_div_second_method(
    cf: ContinuedFraction, n: Nat
) {
    cf.has_valid_prefix(n) implies
        cf.prefix_value(n) = Rat.from_nat(cf.prefix_convergent(n).first) /
            Rat.from_nat(cf.prefix_convergent(n).second)
} by {
    if cf.has_valid_prefix(n) {
        continued_fraction_has_valid_prefix_elim(cf, n)
        continued_fraction_prefix_value_eq_convergent_first_div_second(
            cf.coefficients, n)
        continued_fraction_prefix_value_eq_coefficients_prefix_value(cf, n)
        continued_fraction_prefix_convergent_eq_coefficients_prefix_convergent(cf, n)
        cf.prefix_value(n) = Rat.from_nat(cf.prefix_convergent(n).first) /
            Rat.from_nat(cf.prefix_convergent(n).second)
    }
}
