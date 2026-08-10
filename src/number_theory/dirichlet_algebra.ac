from nat import Nat
from number_theory.arithmetic_functions import nat_dirichlet_unit_fn, is_multiplicative_nat_fn,
    nat_dirichlet_unit_fn_multiplicative
from number_theory.dirichlet import dirichlet_convolve, dirichlet_convolve_comm,
    dirichlet_convolve_unit_right, dirichlet_convolve_unit_left,
    dirichlet_convolve_at_zero, dirichlet_convolve_multiplicative
from number_theory.dirichlet_assoc import dirichlet_convolve_assoc
numerals Nat

/// The Dirichlet convolution algebra: the laws of the Dirichlet convolution
/// `dirichlet_convolve(f, g)(n) = sum_{d | n} f(d) * g(n / d)` on arithmetic
/// functions.  Dirichlet convolution is commutative and associative, and the
/// Dirichlet unit `nat_dirichlet_unit_fn` (one at one, zero elsewhere) is a
/// two-sided identity at every positive argument; on functions vanishing at
/// zero it is a two-sided identity at every argument.  Convolution of
/// multiplicative functions is multiplicative.  The pointwise laws are proved
/// in `number_theory.dirichlet` and `number_theory.dirichlet_assoc`; this file
/// collects them as the convolution algebra and adds the function-level
/// identity laws.

/// Dirichlet convolution is commutative.
theorem dirichlet_convolve_comm_law(f: Nat -> Nat, g: Nat -> Nat) {
    dirichlet_convolve(f, g) = dirichlet_convolve(g, f)
} by {
    dirichlet_convolve_comm(f, g)
}

/// Dirichlet convolution is commutative at every argument.
theorem dirichlet_convolve_comm_pointwise(f: Nat -> Nat, g: Nat -> Nat, n: Nat) {
    dirichlet_convolve(f, g)(n) = dirichlet_convolve(g, f)(n)
} by {
    dirichlet_convolve_comm(f, g)
    dirichlet_convolve(f, g)(n) = dirichlet_convolve(g, f)(n)
}

/// Dirichlet convolution is associative.
theorem dirichlet_convolve_assoc_law(f: Nat -> Nat, g: Nat -> Nat, h: Nat -> Nat) {
    dirichlet_convolve(f, dirichlet_convolve(g, h)) =
        dirichlet_convolve(dirichlet_convolve(f, g), h)
} by {
    dirichlet_convolve_assoc(f, g, h)
}

/// Dirichlet convolution is associative at every argument.
theorem dirichlet_convolve_assoc_pointwise(f: Nat -> Nat, g: Nat -> Nat,
        h: Nat -> Nat, n: Nat) {
    dirichlet_convolve(f, dirichlet_convolve(g, h))(n) =
        dirichlet_convolve(dirichlet_convolve(f, g), h)(n)
} by {
    dirichlet_convolve_assoc(f, g, h)
    dirichlet_convolve(f, dirichlet_convolve(g, h))(n) =
        dirichlet_convolve(dirichlet_convolve(f, g), h)(n)
}

/// The Dirichlet unit is a right identity at positive arguments:
/// `(f * unit)(n) = f(n)` for `n > 0`.
theorem dirichlet_convolve_unit_right_law(f: Nat -> Nat, n: Nat) {
    Nat.0 < n implies dirichlet_convolve(f, nat_dirichlet_unit_fn)(n) = f(n)
} by {
    dirichlet_convolve_unit_right(f, n)
}

/// The Dirichlet unit is a left identity at positive arguments:
/// `(unit * f)(n) = f(n)` for `n > 0`.
theorem dirichlet_convolve_unit_left_law(f: Nat -> Nat, n: Nat) {
    Nat.0 < n implies dirichlet_convolve(nat_dirichlet_unit_fn, f)(n) = f(n)
} by {
    dirichlet_convolve_unit_left(f, n)
}

/// An arithmetic function that vanishes at zero is a fixed point of
/// right convolution with the Dirichlet unit: `f * unit = f`.  Since
/// `dirichlet_convolve(f, g)(0) = 0` always, this is the sharpest function
/// form of the right identity law.
theorem dirichlet_convolve_unit_right_fn(f: Nat -> Nat) {
    f(Nat.0) = Nat.0 implies dirichlet_convolve(f, nat_dirichlet_unit_fn) = f
} by {
    if f(Nat.0) = Nat.0 {
        forall(n: Nat) {
            if n = Nat.0 {
                dirichlet_convolve_at_zero(f, nat_dirichlet_unit_fn)
                dirichlet_convolve(f, nat_dirichlet_unit_fn)(n) = Nat.0
                f(n) = Nat.0
                dirichlet_convolve(f, nat_dirichlet_unit_fn)(n) = f(n)
            } else {
                Nat.0 < n
                dirichlet_convolve_unit_right(f, n)
                dirichlet_convolve(f, nat_dirichlet_unit_fn)(n) = f(n)
            }
        }
    }
}

/// An arithmetic function that vanishes at zero is a fixed point of left
/// convolution with the Dirichlet unit: `unit * f = f`.
theorem dirichlet_convolve_unit_left_fn(f: Nat -> Nat) {
    f(Nat.0) = Nat.0 implies dirichlet_convolve(nat_dirichlet_unit_fn, f) = f
} by {
    if f(Nat.0) = Nat.0 {
        forall(n: Nat) {
            if n = Nat.0 {
                dirichlet_convolve_at_zero(nat_dirichlet_unit_fn, f)
                dirichlet_convolve(nat_dirichlet_unit_fn, f)(n) = Nat.0
                f(n) = Nat.0
                dirichlet_convolve(nat_dirichlet_unit_fn, f)(n) = f(n)
            } else {
                Nat.0 < n
                dirichlet_convolve_unit_left(f, n)
                dirichlet_convolve(nat_dirichlet_unit_fn, f)(n) = f(n)
            }
        }
    }
}

/// The Dirichlet convolution of multiplicative arithmetic functions is
/// multiplicative.
theorem dirichlet_convolve_multiplicative_law(f: Nat -> Nat, g: Nat -> Nat) {
    is_multiplicative_nat_fn(f) and is_multiplicative_nat_fn(g)
        implies is_multiplicative_nat_fn(dirichlet_convolve(f, g))
} by {
    dirichlet_convolve_multiplicative(f, g)
}

/// Convolving a multiplicative function with the Dirichlet unit again gives a
/// multiplicative function.
theorem dirichlet_convolve_unit_multiplicative(f: Nat -> Nat) {
    is_multiplicative_nat_fn(f) implies
        is_multiplicative_nat_fn(dirichlet_convolve(f, nat_dirichlet_unit_fn))
} by {
    if is_multiplicative_nat_fn(f) {
        nat_dirichlet_unit_fn_multiplicative
        is_multiplicative_nat_fn(nat_dirichlet_unit_fn)
        dirichlet_convolve_multiplicative(f, nat_dirichlet_unit_fn)
        is_multiplicative_nat_fn(dirichlet_convolve(f, nat_dirichlet_unit_fn))
    }
}
