/// Irrationality of square and cube roots of two via the rational root
/// theorem: the monic integer polynomials `X^2 - 2` and `X^3 - 2` have no
/// rational root, so no rational number squares or cubes to two.
///
/// The rational root theorem forces a rational root in lowest terms `a / b`
/// of a monic integer polynomial to have denominator one, so the root is an
/// integer; the numerator must then divide the constant coefficient, and a
/// direct check of the finitely many candidates contradicts the vanishing of
/// the value.

from nat import Nat, divides_lte, lte_suc_suc, lte_imp_not_lt, lt_or_lte,
    lt_trans, zero_or_suc, not_lt_zero, lt_suc, trichotomy, lt_cancel_suc, lt_suc_right,
    lte_antisymm, lt_imp_lte_suc, lte_trans, lt_not_ref
from order import lt_imp_lte
from int import Int, abs, div_imp_div_abs, abs_from_nat, abs_neg, neg_or_pos, abs_zero_imp_zero,
    neg_neg, one_pos, pos_is_not_neg, pos_part_from, from_eq_neg_from
from int import mul_nat_from_nat_left, mul_nat_from_nat_right
from algebra.ring.ring import mul_neg_neg
from rat import Rat, denom_nonzero, denom_positive, div_from_int, mul_div_cancels,
    from_int_num, from_int_denom, is_reduced
from data.int.int_coprime import is_coprime
from data.rat.rat_int_hom import from_int_add, from_int_mul, from_int_one, int_to_rat_hom,
    int_to_rat_hom_apply, int_zero_iff_rat_zero
from polynomial import Polynomial, polynomial_constant, polynomial_monomial, polynomial_eval,
    polynomial_mul, polynomial_support_bounded_by, polynomial_eval_add, polynomial_eval_constant,
    polynomial_eval_bound, polynomial_eval_bound_eq_coeff_eval,
    polynomial_eval_eq_eval_bound_of_support_bounded, polynomial_eval_map,
    polynomial_support_bounded_by_monotone, polynomial_add_coeff, coeff_eval, coeff_tail,
    polynomial_monomial_coeff_self, polynomial_monomial_coeff_of_ne,
    polynomial_constant_coeff_zero, polynomial_constant_coeff_of_ne_zero, polynomial_map,
    polynomial_constant_support_bounded_by_one, polynomial_add_support_bounded_by
from complex import polynomial_monomial_support_bounded_by_suc
from polynomial_int_roots import rationalise, rationalise_support_bounded_by
from number_theory.polynomial_nt import int_quadratic_polynomial, int_quadratic_coeff_zero,
    int_quadratic_coeff_two, int_quadratic_support_bounded_by_three,
    quadratic_rational_root_numerator_divides, quadratic_rational_root_denominator_divides,
    polynomial_rational_root_numerator_divides, polynomial_rational_root_denominator_divides

numerals Nat
numerals Int

/// An integer divisor of minus two is one of `+-1, +-2`.
theorem int_divisor_of_neg_two(a: Int) {
    a.divides(-(Int.2)) implies a = Int.1 or a = -(Int.1) or a = Int.2 or a = -(Int.2)
} by {
    if a.divides(-(Int.2)) {
        div_imp_div_abs(a, -(Int.2))
        abs(a).divides(abs(-(Int.2)))
        abs_neg(Int.2)
        abs(-(Int.2)) = abs(Int.2)
        abs_from_nat(Nat.2)
        abs(Int.2) = Nat.2
        abs(a).divides(Nat.2)
        divides_lte(abs(a), Nat.2)
        abs(a) = Nat.0 or abs(a) <= Nat.2
        if abs(a) = Nat.0 {
            abs_zero_imp_zero(a)
            a = Int.0
            if a = Int.0 {
                a.divides(-(Int.2)) = exists(c: Int) {
                    c * a = -(Int.2)
                }
                exists(c: Int) {
                    c * a = -(Int.2)
                }
                let (c: Int) satisfy {
                    c * a = -(Int.2)
                }
                a = Int.0
                c * a = c * Int.0
                c * Int.0 = Int.0
                c * a = Int.0
                -(Int.2) = Int.0
                false
            }
            false
        }
        abs(a) != Nat.0
        abs(a) <= Nat.2
        trichotomy(abs(a), Nat.2)
        if abs(a) < Nat.2 {
            zero_or_suc(abs(a))
            if abs(a) = Nat.0 {
                false
            }
            let ap: Nat satisfy {
                abs(a) = ap.suc
            }
            abs(a) = ap.suc
            ap.suc < Nat.2
            lt_cancel_suc(ap, Nat.1)
            ap < Nat.1
            lt_suc_right(ap, Nat.0)
            if ap < Nat.0 {
                not_lt_zero(ap)
                false
            }
            ap = Nat.0
            abs(a) = Nat.1
            neg_or_pos(a)
            if a = Int.from_nat(abs(a)) {
                abs(a) = Nat.1
                a = Int.from_nat(Nat.1)
                a = Int.1
            } else {
                a = -(Int.from_nat(abs(a)))
                abs(a) = Nat.1
                a = -(Int.from_nat(Nat.1))
                a = -(Int.1)
            }
            a = Int.1 or a = -(Int.1) or a = Int.2 or a = -(Int.2)
        }
        if Nat.2 < abs(a) {
            lte_imp_not_lt(abs(a), Nat.2)
            not Nat.2 < abs(a)
            false
        }
        if abs(a) = Nat.2 {
            neg_or_pos(a)
            if a = Int.from_nat(abs(a)) {
                abs(a) = Nat.2
                a = Int.from_nat(Nat.2)
                a = Int.2
            } else {
                a = -(Int.from_nat(abs(a)))
                abs(a) = Nat.2
                a = -(Int.from_nat(Nat.2))
                a = -(Int.2)
            }
            a = Int.1 or a = -(Int.1) or a = Int.2 or a = -(Int.2)
        }
        a = Int.1 or a = -(Int.1) or a = Int.2 or a = -(Int.2)
    }
}

/// A positive integer divisor of one is one.
theorem int_positive_divisor_of_one(b: Int) {
    b.divides(Int.1) and b.is_positive implies b = Int.1
} by {
    if b.divides(Int.1) and b.is_positive {
        div_imp_div_abs(b, Int.1)
        abs(b).divides(abs(Int.1))
        abs_from_nat(Nat.1)
        abs(Int.1) = Nat.1
        abs(b).divides(Nat.1)
        divides_lte(abs(b), Nat.1)
        abs(b) = Nat.0 or abs(b) <= Nat.1
        if abs(b) = Nat.0 {
            abs_zero_imp_zero(b)
            b = Int.0
            if b = Int.0 {
                b.divides(Int.1) = exists(c: Int) {
                    c * b = Int.1
                }
                exists(c: Int) {
                    c * b = Int.1
                }
                let (c: Int) satisfy {
                    c * b = Int.1
                }
                b = Int.0
                c * b = c * Int.0
                c * Int.0 = Int.0
                c * b = Int.0
                Int.1 = Int.0
                false
            }
            false
        }
        abs(b) != Nat.0
        abs(b) <= Nat.1
        if abs(b) = Nat.1 {
            neg_or_pos(b)
            if b = Int.from_nat(abs(b)) {
                abs(b) = Nat.1
                b = Int.from_nat(Nat.1)
                b = Int.1
            } else {
                b = -(Int.from_nat(abs(b)))
                abs(b) = Nat.1
                b = -(Int.from_nat(Nat.1))
                b = -(Int.1)
                if b.is_positive {
                    b = -(Int.1)
                    b.is_positive = (-b).is_negative
                    -b = -(-(Int.1))
                    neg_neg(Int.1)
                    -(-(Int.1)) = Int.1
                    -b = Int.1
                    Int.1.is_negative
                    one_pos
                    Int.1.is_positive
                    pos_is_not_neg(Int.1)
                    not Int.1.is_negative
                    false
                }
                false
            }
            b = Int.1
        } else {
            if abs(b) = Nat.0 {
                false
            }
            lt_or_lte(abs(b), Nat.1)
            if abs(b) < Nat.1 {
                lt_imp_lte(abs(b), Nat.0)
                abs(b) <= Nat.0
                if abs(b) = Nat.0 {
                    false
                }
                zero_or_suc(abs(b))
                let bp: Nat satisfy {
                    abs(b) = bp.suc
                }
                abs(b) = bp.suc
                lte_suc_suc(bp, Nat.0)
                bp.suc <= Nat.1
                abs(b) <= Nat.1
                lte_imp_not_lt(abs(b), Nat.1)
                not abs(b) < Nat.1
                false
            }
            if Nat.1 <= abs(b) {
                false
            }
            false
        }
        b = Int.1
    }
}

/// Evaluation commutes with rationalising: the value of the rationalised
/// polynomial at the image of an integer is the image of the integer value.
theorem rationalise_eval_int(p: Polynomial[Int], x: Int) {
    polynomial_eval(rationalise(p), Rat.from_int(x)) = Rat.from_int(polynomial_eval(p, x))
} by {
    polynomial_eval_map(int_to_rat_hom, p, x)
    polynomial_eval(polynomial_map(int_to_rat_hom, p), int_to_rat_hom.hom(x)) =
        int_to_rat_hom.hom(polynomial_eval(p, x))
    int_to_rat_hom_apply(x)
    int_to_rat_hom.hom(x) = Rat.from_int(x)
    int_to_rat_hom_apply(polynomial_eval(p, x))
    int_to_rat_hom.hom(polynomial_eval(p, x)) = Rat.from_int(polynomial_eval(p, x))
    rationalise(p) = polynomial_map(int_to_rat_hom, p)
    polynomial_eval(rationalise(p), Rat.from_int(x)) = Rat.from_int(polynomial_eval(p, x))
}

/// A rational root of the rationalised polynomial is a root of the original.
theorem rationalise_root_imp_int_root(p: Polynomial[Int], x: Int) {
    polynomial_eval(rationalise(p), Rat.from_int(x)) = Rat.0 implies
    polynomial_eval(p, x) = Int.0
} by {
    if polynomial_eval(rationalise(p), Rat.from_int(x)) = Rat.0 {
        rationalise_eval_int(p, x)
        polynomial_eval(rationalise(p), Rat.from_int(x)) = Rat.from_int(polynomial_eval(p, x))
        Rat.from_int(polynomial_eval(p, x)) = Rat.0
        int_zero_iff_rat_zero(polynomial_eval(p, x))
        polynomial_eval(p, x) = Int.0
    }
}

/// The polynomial `X^2 - 2` with rational coefficients.
let rat_x_squared_minus_two: Polynomial[Rat] =
    rationalise(int_quadratic_polynomial(Int.1, Int.0, -(Int.2)))

/// The linear coefficient of an integer quadratic is its middle term.
theorem int_quadratic_coeff_one_local(a: Int, b: Int, c: Int) {
    int_quadratic_polynomial(a, b, c).coeff(Nat.1) = b
} by {
    polynomial_add_coeff(polynomial_constant(c) + polynomial_monomial(Nat.1, b),
        polynomial_monomial(Nat.2, a), Nat.1)
    int_quadratic_polynomial(a, b, c).coeff(Nat.1) =
        (polynomial_constant(c) + polynomial_monomial(Nat.1, b)).coeff(Nat.1) +
        polynomial_monomial(Nat.2, a).coeff(Nat.1)
    polynomial_add_coeff(polynomial_constant(c), polynomial_monomial(Nat.1, b), Nat.1)
    (polynomial_constant(c) + polynomial_monomial(Nat.1, b)).coeff(Nat.1) =
        polynomial_constant(c).coeff(Nat.1) + polynomial_monomial(Nat.1, b).coeff(Nat.1)
    Nat.1 != Nat.0
    polynomial_constant_coeff_of_ne_zero(c, Nat.1)
    polynomial_constant(c).coeff(Nat.1) = Int.0
    polynomial_monomial_coeff_self(Nat.1, b)
    polynomial_monomial(Nat.1, b).coeff(Nat.1) = b
    (polynomial_constant(c) + polynomial_monomial(Nat.1, b)).coeff(Nat.1) = Int.0 + b
    Int.0 + b = b
    Nat.1 != Nat.2
    polynomial_monomial_coeff_of_ne[Int](Nat.2, a, Nat.1)
    polynomial_monomial(Nat.2, a).coeff(Nat.1) = Int.0
    int_quadratic_polynomial(a, b, c).coeff(Nat.1) = b + Int.0
    b + Int.0 = b
}

/// The value of the integer quadratic `X^2 - 2` at an integer is `x^2 - 2`.
theorem int_x_squared_minus_two_eval(x: Int) {
    polynomial_eval(int_quadratic_polynomial(Int.1, Int.0, -(Int.2)), x) = x * x - Int.2
} by {
    int_quadratic_support_bounded_by_three(Int.1, Int.0, -(Int.2))
    polynomial_support_bounded_by(int_quadratic_polynomial(Int.1, Int.0, -(Int.2)),
        Nat.0.suc.suc.suc)
    polynomial_eval_eq_eval_bound_of_support_bounded(int_quadratic_polynomial(Int.1, Int.0, -(Int.2)),
        x, Nat.0.suc.suc.suc)
    polynomial_eval(int_quadratic_polynomial(Int.1, Int.0, -(Int.2)), x) =
        polynomial_eval_bound(int_quadratic_polynomial(Int.1, Int.0, -(Int.2)), x, Nat.0.suc.suc.suc)
    polynomial_eval_bound_eq_coeff_eval(int_quadratic_polynomial(Int.1, Int.0, -(Int.2)),
        x, Nat.0.suc.suc.suc)
    polynomial_eval_bound(int_quadratic_polynomial(Int.1, Int.0, -(Int.2)), x, Nat.0.suc.suc.suc) =
        coeff_eval(int_quadratic_polynomial(Int.1, Int.0, -(Int.2)).coeff, x, Nat.0.suc.suc.suc)
    coeff_eval(int_quadratic_polynomial(Int.1, Int.0, -(Int.2)).coeff, x, Nat.0.suc.suc.suc) =
        int_quadratic_polynomial(Int.1, Int.0, -(Int.2)).coeff(Nat.0) +
        x * coeff_eval(coeff_tail(int_quadratic_polynomial(Int.1, Int.0, -(Int.2)).coeff), x,
            Nat.0.suc.suc)
    coeff_eval(coeff_tail(int_quadratic_polynomial(Int.1, Int.0, -(Int.2)).coeff), x,
        Nat.0.suc.suc) =
        coeff_tail(int_quadratic_polynomial(Int.1, Int.0, -(Int.2)).coeff, Nat.0) +
        x * coeff_eval(coeff_tail(coeff_tail(
            int_quadratic_polynomial(Int.1, Int.0, -(Int.2)).coeff)), x, Nat.0.suc)
    coeff_tail(int_quadratic_polynomial(Int.1, Int.0, -(Int.2)).coeff, Nat.0) =
        int_quadratic_polynomial(Int.1, Int.0, -(Int.2)).coeff(Nat.1)
    coeff_eval(coeff_tail(coeff_tail(
        int_quadratic_polynomial(Int.1, Int.0, -(Int.2)).coeff)), x, Nat.0.suc) =
        coeff_tail(coeff_tail(int_quadratic_polynomial(Int.1, Int.0, -(Int.2)).coeff), Nat.0) +
        x * coeff_eval(coeff_tail(coeff_tail(coeff_tail(
            int_quadratic_polynomial(Int.1, Int.0, -(Int.2)).coeff))), x, Nat.0)
    coeff_tail(coeff_tail(int_quadratic_polynomial(Int.1, Int.0, -(Int.2)).coeff), Nat.0) =
        int_quadratic_polynomial(Int.1, Int.0, -(Int.2)).coeff(Nat.2)
    coeff_eval(coeff_tail(coeff_tail(coeff_tail(
        int_quadratic_polynomial(Int.1, Int.0, -(Int.2)).coeff))), x, Nat.0) = Int.0
    int_quadratic_coeff_zero(Int.1, Int.0, -(Int.2))
    int_quadratic_polynomial(Int.1, Int.0, -(Int.2)).coeff(Nat.0) = -(Int.2)
    int_quadratic_coeff_one_local(Int.1, Int.0, -(Int.2))
    int_quadratic_polynomial(Int.1, Int.0, -(Int.2)).coeff(Nat.1) = Int.0
    int_quadratic_coeff_two(Int.1, Int.0, -(Int.2))
    int_quadratic_polynomial(Int.1, Int.0, -(Int.2)).coeff(Nat.2) = Int.1
    coeff_eval(int_quadratic_polynomial(Int.1, Int.0, -(Int.2)).coeff, x, Nat.0.suc.suc.suc) =
        -(Int.2) + x * (Int.0 + x * (Int.1 + x * Int.0))
    -(Int.2) + x * (Int.0 + x * (Int.1 + x * Int.0)) = x * x - Int.2
    polynomial_eval(int_quadratic_polynomial(Int.1, Int.0, -(Int.2)), x) = x * x - Int.2
}

/// `X^2 - 2` has no rational root.
theorem rat_x_squared_minus_two_has_no_root {
    not exists(r: Rat) {
        polynomial_eval(rat_x_squared_minus_two, r) = Rat.0
    }
} by {
    if exists(r: Rat) {
        polynomial_eval(rat_x_squared_minus_two, r) = Rat.0
    } {
        let (r: Rat) satisfy {
            polynomial_eval(rat_x_squared_minus_two, r) = Rat.0
        }
        div_from_int(r)
        Rat.from_int(r.num) / Rat.from_int(r.denom) = r
        polynomial_eval(rat_x_squared_minus_two,
            Rat.from_int(r.num) / Rat.from_int(r.denom)) = Rat.0
        int_quadratic_support_bounded_by_three(Int.1, Int.0, -(Int.2))
        polynomial_support_bounded_by(int_quadratic_polynomial(Int.1, Int.0, -(Int.2)),
            Nat.0.suc.suc.suc)
        denom_nonzero(r)
        r.denom != Int.0
        is_reduced(r.num, r.denom)
        r.num.gcd(r.denom) = Int.1
        is_coprime(r.num, r.denom) = (r.num.gcd(r.denom) = Int.1)
        is_coprime(r.num, r.denom)
        quadratic_rational_root_numerator_divides(Int.1, Int.0, -(Int.2), r.num, r.denom)
        r.num.divides(-(Int.2))
        quadratic_rational_root_denominator_divides(Int.1, Int.0, -(Int.2), r.num, r.denom)
        r.denom.divides(Int.1)
        denom_positive(r)
        r.denom.is_positive
        int_positive_divisor_of_one(r.denom)
        r.denom = Int.1
        from_int_denom(r.num)
        Rat.from_int(r.num).denom = Int.1
        Rat.from_int(r.num) / Rat.from_int(Int.1) = Rat.from_int(r.num)
        from_int_one
        Rat.from_int(Int.1) = Rat.1
        r.denom = Int.1
        Rat.from_int(r.num) / Rat.from_int(r.denom) = Rat.from_int(r.num) / Rat.1
        Rat.from_int(r.num) / Rat.1 = Rat.from_int(r.num)
        Rat.from_int(r.num) / Rat.from_int(r.denom) = Rat.from_int(r.num)
        r = Rat.from_int(r.num)
        polynomial_eval(rat_x_squared_minus_two, Rat.from_int(r.num)) = Rat.0
        rationalise_root_imp_int_root(int_quadratic_polynomial(Int.1, Int.0, -(Int.2)), r.num)
        polynomial_eval(int_quadratic_polynomial(Int.1, Int.0, -(Int.2)), r.num) = Int.0
        int_x_squared_minus_two_eval(r.num)
        polynomial_eval(int_quadratic_polynomial(Int.1, Int.0, -(Int.2)), r.num) =
            r.num * r.num - Int.2
        r.num * r.num - Int.2 = Int.0
        r.num * r.num = Int.2
        int_divisor_of_neg_two(r.num)
        if r.num = Int.1 {
            r.num * r.num = Int.1 * Int.1
            Int.1 * Int.1 = Int.1
            r.num * r.num = Int.1
            r.num * r.num = Int.2
            Int.1 = Int.2
            Int.1 = Int.from_nat(Nat.1)
            Int.2 = Int.from_nat(Nat.2)
            Int.from_nat(Nat.1) = Int.from_nat(Nat.2)
            pos_part_from(Nat.1)
            Int.from_nat(Nat.1).pos_part = Nat.1
            pos_part_from(Nat.2)
            Int.from_nat(Nat.2).pos_part = Nat.2
            Nat.1 = Nat.2
            false
        }
        if r.num = -(Int.1) {
            r.num * r.num = (-(Int.1)) * (-(Int.1))
            mul_neg_neg(Int.1, Int.1)
            (-(Int.1)) * (-(Int.1)) = Int.1 * Int.1
            Int.1 * Int.1 = Int.1
            r.num * r.num = Int.1
            r.num * r.num = Int.2
            Int.1 = Int.2
            Int.1 = Int.from_nat(Nat.1)
            Int.2 = Int.from_nat(Nat.2)
            Int.from_nat(Nat.1) = Int.from_nat(Nat.2)
            pos_part_from(Nat.1)
            Int.from_nat(Nat.1).pos_part = Nat.1
            pos_part_from(Nat.2)
            Int.from_nat(Nat.2).pos_part = Nat.2
            Nat.1 = Nat.2
            false
        }
        if r.num = Int.2 {
            r.num * r.num = Int.2 * Int.2
            Int.2 = Int.from_nat(Nat.2)
            Int.4 = Int.from_nat(Nat.4)
            Int.2 * Int.2 = Int.from_nat(Nat.2) * Int.from_nat(Nat.2)
            mul_nat_from_nat_right(Int.from_nat(Nat.2), Nat.2)
            Int.from_nat(Nat.2).mul_nat(Nat.2) = Int.from_nat(Nat.2) * Int.from_nat(Nat.2)
            mul_nat_from_nat_left(Nat.2, Nat.2)
            Int.from_nat(Nat.2).mul_nat(Nat.2) = Int.from_nat(Nat.2 * Nat.2)
            Nat.2 * Nat.2 = Nat.4
            Int.from_nat(Nat.2) * Int.from_nat(Nat.2) = Int.from_nat(Nat.4)
            Int.2 * Int.2 = Int.4
            r.num * r.num = Int.4
            r.num * r.num = Int.2
            Int.4 = Int.2
            Int.4 = Int.from_nat(Nat.4)
            Int.2 = Int.from_nat(Nat.2)
            Int.from_nat(Nat.4) = Int.from_nat(Nat.2)
            pos_part_from(Nat.4)
            Int.from_nat(Nat.4).pos_part = Nat.4
            pos_part_from(Nat.2)
            Int.from_nat(Nat.2).pos_part = Nat.2
            Nat.4 = Nat.2
            false
        }
        if r.num = -(Int.2) {
            r.num * r.num = (-(Int.2)) * (-(Int.2))
            mul_neg_neg(Int.2, Int.2)
            (-(Int.2)) * (-(Int.2)) = Int.2 * Int.2
            Int.2 = Int.from_nat(Nat.2)
            Int.4 = Int.from_nat(Nat.4)
            Int.2 * Int.2 = Int.from_nat(Nat.2) * Int.from_nat(Nat.2)
            mul_nat_from_nat_right(Int.from_nat(Nat.2), Nat.2)
            Int.from_nat(Nat.2).mul_nat(Nat.2) = Int.from_nat(Nat.2) * Int.from_nat(Nat.2)
            mul_nat_from_nat_left(Nat.2, Nat.2)
            Int.from_nat(Nat.2).mul_nat(Nat.2) = Int.from_nat(Nat.2 * Nat.2)
            Nat.2 * Nat.2 = Nat.4
            Int.from_nat(Nat.2) * Int.from_nat(Nat.2) = Int.from_nat(Nat.4)
            Int.2 * Int.2 = Int.4
            r.num * r.num = Int.4
            r.num * r.num = Int.2
            Int.4 = Int.2
            Int.4 = Int.from_nat(Nat.4)
            Int.2 = Int.from_nat(Nat.2)
            Int.from_nat(Nat.4) = Int.from_nat(Nat.2)
            pos_part_from(Nat.4)
            Int.from_nat(Nat.4).pos_part = Nat.4
            pos_part_from(Nat.2)
            Int.from_nat(Nat.2).pos_part = Nat.2
            Nat.4 = Nat.2
            false
        }
        r.num != Int.1
        r.num != -(Int.1)
        r.num != Int.2
        r.num != -(Int.2)
        false
    }
}

/// The polynomial `X^3 - 2` with integer coefficients.
let int_cubic_minus_two: Polynomial[Int] =
    polynomial_constant(-(Int.2)) + polynomial_monomial(Nat.3, Int.1)

/// `X^3 - 2` is supported below four coefficient slots.
theorem int_cubic_minus_two_support_bounded {
    polynomial_support_bounded_by(int_cubic_minus_two, Nat.0.suc.suc.suc.suc)
} by {
    polynomial_constant_support_bounded_by_one(-(Int.2))
    polynomial_support_bounded_by(polynomial_constant(-(Int.2)), Nat.0.suc)
    lt_suc(Nat.3)
    Nat.3 < Nat.3.suc
    lt_imp_lte[Nat](Nat.3, Nat.3.suc)
    Nat.3 <= Nat.3.suc
    lt_suc(Nat.1)
    Nat.1 < Nat.1.suc
    lt_imp_lte[Nat](Nat.1, Nat.1.suc)
    Nat.1 <= Nat.2
    lt_suc(Nat.2)
    Nat.2 < Nat.2.suc
    lt_imp_lte[Nat](Nat.2, Nat.2.suc)
    Nat.2 <= Nat.3
    lte_trans(Nat.1, Nat.2, Nat.3)
    Nat.1 <= Nat.3
    Nat.3.suc = Nat.4
    Nat.1 <= Nat.4
    polynomial_support_bounded_by_monotone(polynomial_constant(-(Int.2)), Nat.1, Nat.4)
    polynomial_support_bounded_by(polynomial_constant(-(Int.2)), Nat.4)
    polynomial_monomial_support_bounded_by_suc[Int](Nat.3, Int.1)
    polynomial_support_bounded_by(polynomial_monomial(Nat.3, Int.1), Nat.3.suc)
    Nat.3.suc = Nat.4
    polynomial_support_bounded_by(polynomial_monomial(Nat.3, Int.1), Nat.4)
    polynomial_add_support_bounded_by(polynomial_constant(-(Int.2)),
        polynomial_monomial(Nat.3, Int.1), Nat.4)
    polynomial_support_bounded_by(polynomial_constant(-(Int.2)) + polynomial_monomial(Nat.3, Int.1),
        Nat.4)
    polynomial_support_bounded_by(int_cubic_minus_two, Nat.4)
    polynomial_support_bounded_by(int_cubic_minus_two, Nat.0.suc.suc.suc.suc)
}

/// The constant coefficient of `X^3 - 2` is minus two.
theorem int_cubic_minus_two_coeff_zero {
    int_cubic_minus_two.coeff(Nat.0) = -(Int.2)
} by {
    polynomial_add_coeff(polynomial_constant(-(Int.2)), polynomial_monomial(Nat.3, Int.1), Nat.0)
    int_cubic_minus_two.coeff(Nat.0) =
        polynomial_constant(-(Int.2)).coeff(Nat.0) + polynomial_monomial(Nat.3, Int.1).coeff(Nat.0)
    polynomial_constant_coeff_zero(-(Int.2))
    polynomial_constant(-(Int.2)).coeff(Nat.0) = -(Int.2)
    Nat.0 != Nat.3
    polynomial_monomial_coeff_of_ne[Int](Nat.3, Int.1, Nat.0)
    polynomial_monomial(Nat.3, Int.1).coeff(Nat.0) = Int.0
    int_cubic_minus_two.coeff(Nat.0) = -(Int.2) + Int.0
    -(Int.2) + Int.0 = -(Int.2)
}

/// The leading coefficient of `X^3 - 2` is one.
theorem int_cubic_minus_two_coeff_three {
    int_cubic_minus_two.coeff(Nat.3) = Int.1
} by {
    polynomial_add_coeff(polynomial_constant(-(Int.2)), polynomial_monomial(Nat.3, Int.1), Nat.3)
    int_cubic_minus_two.coeff(Nat.3) =
        polynomial_constant(-(Int.2)).coeff(Nat.3) + polynomial_monomial(Nat.3, Int.1).coeff(Nat.3)
    Nat.3 != Nat.0
    polynomial_constant_coeff_of_ne_zero(-(Int.2), Nat.3)
    polynomial_constant(-(Int.2)).coeff(Nat.3) = Int.0
    polynomial_monomial_coeff_self(Nat.3, Int.1)
    polynomial_monomial(Nat.3, Int.1).coeff(Nat.3) = Int.1
    int_cubic_minus_two.coeff(Nat.3) = Int.0 + Int.1
    Int.0 + Int.1 = Int.1
}

/// The value of `X^3 - 2` at an integer is `x^3 - 2`.
theorem int_cubic_minus_two_eval(x: Int) {
    polynomial_eval(int_cubic_minus_two, x) = x * (x * x) - Int.2
} by {
    int_cubic_minus_two_support_bounded
    polynomial_support_bounded_by(int_cubic_minus_two, Nat.0.suc.suc.suc.suc)
    polynomial_eval_eq_eval_bound_of_support_bounded(int_cubic_minus_two, x,
        Nat.0.suc.suc.suc.suc)
    polynomial_eval(int_cubic_minus_two, x) =
        polynomial_eval_bound(int_cubic_minus_two, x, Nat.0.suc.suc.suc.suc)
    polynomial_eval_bound_eq_coeff_eval(int_cubic_minus_two, x, Nat.0.suc.suc.suc.suc)
    polynomial_eval_bound(int_cubic_minus_two, x, Nat.0.suc.suc.suc.suc) =
        coeff_eval(int_cubic_minus_two.coeff, x, Nat.0.suc.suc.suc.suc)
    coeff_eval(int_cubic_minus_two.coeff, x, Nat.0.suc.suc.suc.suc) =
        int_cubic_minus_two.coeff(Nat.0) +
        x * coeff_eval(coeff_tail(int_cubic_minus_two.coeff), x, Nat.0.suc.suc.suc)
    coeff_eval(coeff_tail(int_cubic_minus_two.coeff), x, Nat.0.suc.suc.suc) =
        coeff_tail(int_cubic_minus_two.coeff, Nat.0) +
        x * coeff_eval(coeff_tail(coeff_tail(int_cubic_minus_two.coeff)), x, Nat.0.suc.suc)
    coeff_tail(int_cubic_minus_two.coeff, Nat.0) = int_cubic_minus_two.coeff(Nat.1)
    coeff_eval(coeff_tail(coeff_tail(int_cubic_minus_two.coeff)), x, Nat.0.suc.suc) =
        coeff_tail(coeff_tail(int_cubic_minus_two.coeff), Nat.0) +
        x * coeff_eval(coeff_tail(coeff_tail(coeff_tail(int_cubic_minus_two.coeff))), x, Nat.0.suc)
    coeff_tail(coeff_tail(int_cubic_minus_two.coeff), Nat.0) =
        int_cubic_minus_two.coeff(Nat.2)
    coeff_eval(coeff_tail(coeff_tail(coeff_tail(int_cubic_minus_two.coeff))), x, Nat.0.suc) =
        coeff_tail(coeff_tail(coeff_tail(int_cubic_minus_two.coeff)), Nat.0) +
        x * coeff_eval(coeff_tail(coeff_tail(coeff_tail(coeff_tail(
            int_cubic_minus_two.coeff)))), x, Nat.0)
    coeff_tail(coeff_tail(coeff_tail(int_cubic_minus_two.coeff)), Nat.0) =
        coeff_tail(coeff_tail(int_cubic_minus_two.coeff), Nat.1)
    coeff_tail(coeff_tail(int_cubic_minus_two.coeff), Nat.1) =
        coeff_tail(int_cubic_minus_two.coeff, Nat.2)
    coeff_tail(int_cubic_minus_two.coeff, Nat.2) =
        int_cubic_minus_two.coeff(Nat.3)
    coeff_eval(coeff_tail(coeff_tail(coeff_tail(coeff_tail(
        int_cubic_minus_two.coeff)))), x, Nat.0) = Int.0
    int_cubic_minus_two_coeff_zero
    int_cubic_minus_two.coeff(Nat.0) = -(Int.2)
    polynomial_add_coeff(polynomial_constant(-(Int.2)), polynomial_monomial(Nat.3, Int.1), Nat.1)
    int_cubic_minus_two.coeff(Nat.1) =
        polynomial_constant(-(Int.2)).coeff(Nat.1) + polynomial_monomial(Nat.3, Int.1).coeff(Nat.1)
    Nat.1 != Nat.0
    polynomial_constant_coeff_of_ne_zero(-(Int.2), Nat.1)
    polynomial_constant(-(Int.2)).coeff(Nat.1) = Int.0
    Nat.1 != Nat.3
    polynomial_monomial_coeff_of_ne[Int](Nat.3, Int.1, Nat.1)
    polynomial_monomial(Nat.3, Int.1).coeff(Nat.1) = Int.0
    int_cubic_minus_two.coeff(Nat.1) = Int.0 + Int.0
    Int.0 + Int.0 = Int.0
    polynomial_add_coeff(polynomial_constant(-(Int.2)), polynomial_monomial(Nat.3, Int.1), Nat.2)
    int_cubic_minus_two.coeff(Nat.2) =
        polynomial_constant(-(Int.2)).coeff(Nat.2) + polynomial_monomial(Nat.3, Int.1).coeff(Nat.2)
    Nat.2 != Nat.0
    polynomial_constant_coeff_of_ne_zero(-(Int.2), Nat.2)
    polynomial_constant(-(Int.2)).coeff(Nat.2) = Int.0
    Nat.2 != Nat.3
    polynomial_monomial_coeff_of_ne[Int](Nat.3, Int.1, Nat.2)
    polynomial_monomial(Nat.3, Int.1).coeff(Nat.2) = Int.0
    int_cubic_minus_two.coeff(Nat.2) = Int.0 + Int.0
    Int.0 + Int.0 = Int.0
    int_cubic_minus_two_coeff_three
    int_cubic_minus_two.coeff(Nat.3) = Int.1
    coeff_eval(int_cubic_minus_two.coeff, x, Nat.0.suc.suc.suc.suc) =
        -(Int.2) + x * (Int.0 + x * (Int.0 + x * (Int.1 + x * Int.0)))
    -(Int.2) + x * (Int.0 + x * (Int.0 + x * (Int.1 + x * Int.0))) = x * (x * x) - Int.2
    polynomial_eval(int_cubic_minus_two, x) = x * (x * x) - Int.2
}

/// The polynomial `X^3 - 2` with rational coefficients.
let rat_x_cubed_minus_two: Polynomial[Rat] = rationalise(int_cubic_minus_two)

/// `X^3 - 2` has no rational root.
theorem rat_x_cubed_minus_two_has_no_root {
    not exists(r: Rat) {
        polynomial_eval(rat_x_cubed_minus_two, r) = Rat.0
    }
} by {
    if exists(r: Rat) {
        polynomial_eval(rat_x_cubed_minus_two, r) = Rat.0
    } {
        let (r: Rat) satisfy {
            polynomial_eval(rat_x_cubed_minus_two, r) = Rat.0
        }
        div_from_int(r)
        Rat.from_int(r.num) / Rat.from_int(r.denom) = r
        polynomial_eval(rat_x_cubed_minus_two, Rat.from_int(r.num) / Rat.from_int(r.denom)) = Rat.0
        int_cubic_minus_two_support_bounded
        polynomial_support_bounded_by(int_cubic_minus_two, Nat.0.suc.suc.suc.suc)
        denom_nonzero(r)
        r.denom != Int.0
        is_reduced(r.num, r.denom)
        r.num.gcd(r.denom) = Int.1
        is_coprime(r.num, r.denom) = (r.num.gcd(r.denom) = Int.1)
        is_coprime(r.num, r.denom)
        Nat.3.suc = Nat.0.suc.suc.suc.suc
        polynomial_support_bounded_by(int_cubic_minus_two, Nat.3.suc)
        polynomial_eval(rationalise(int_cubic_minus_two),
            Rat.from_int(r.num) / Rat.from_int(r.denom)) = Rat.0
        polynomial_rational_root_numerator_divides(int_cubic_minus_two, r.num, r.denom, Nat.3)
        r.num.divides(int_cubic_minus_two.coeff(Nat.0))
        int_cubic_minus_two_coeff_zero
        int_cubic_minus_two.coeff(Nat.0) = -(Int.2)
        r.num.divides(-(Int.2))
        polynomial_rational_root_denominator_divides(int_cubic_minus_two, r.num, r.denom, Nat.3)
        r.denom.divides(int_cubic_minus_two.coeff(Nat.3))
        int_cubic_minus_two_coeff_three
        int_cubic_minus_two.coeff(Nat.3) = Int.1
        r.denom.divides(Int.1)
        denom_positive(r)
        r.denom.is_positive
        int_positive_divisor_of_one(r.denom)
        r.denom = Int.1
        Rat.from_int(r.num) / Rat.from_int(Int.1) = Rat.from_int(r.num)
        from_int_one
        Rat.from_int(Int.1) = Rat.1
        r.denom = Int.1
        Rat.from_int(r.num) / Rat.from_int(r.denom) = Rat.from_int(r.num) / Rat.1
        Rat.from_int(r.num) / Rat.1 = Rat.from_int(r.num)
        Rat.from_int(r.num) / Rat.from_int(r.denom) = Rat.from_int(r.num)
        r = Rat.from_int(r.num)
        polynomial_eval(rat_x_cubed_minus_two, Rat.from_int(r.num)) = Rat.0
        rationalise_root_imp_int_root(int_cubic_minus_two, r.num)
        polynomial_eval(int_cubic_minus_two, r.num) = Int.0
        int_cubic_minus_two_eval(r.num)
        polynomial_eval(int_cubic_minus_two, r.num) = r.num * (r.num * r.num) - Int.2
        r.num * (r.num * r.num) - Int.2 = Int.0
        r.num * (r.num * r.num) = Int.2
        int_divisor_of_neg_two(r.num)
        if r.num = Int.1 {
            r.num * (r.num * r.num) = Int.1 * (Int.1 * Int.1)
            Int.1 * Int.1 = Int.1
            Int.1 * (Int.1 * Int.1) = Int.1
            r.num * (r.num * r.num) = Int.1
            r.num * (r.num * r.num) = Int.2
            Int.1 = Int.2
            Int.1 = Int.from_nat(Nat.1)
            Int.2 = Int.from_nat(Nat.2)
            Int.from_nat(Nat.1) = Int.from_nat(Nat.2)
            pos_part_from(Nat.1)
            Int.from_nat(Nat.1).pos_part = Nat.1
            pos_part_from(Nat.2)
            Int.from_nat(Nat.2).pos_part = Nat.2
            Nat.1 = Nat.2
            false
        }
        if r.num = -(Int.1) {
            r.num * (r.num * r.num) = (-(Int.1)) * ((-(Int.1)) * (-(Int.1)))
            mul_neg_neg(Int.1, Int.1)
            (-(Int.1)) * (-(Int.1)) = Int.1 * Int.1
            Int.1 * Int.1 = Int.1
            (-(Int.1)) * ((-(Int.1)) * (-(Int.1))) = (-(Int.1)) * Int.1
            (-(Int.1)) * Int.1 = -(Int.1 * Int.1)
            Int.1 * Int.1 = Int.1
            (-(Int.1)) * Int.1 = -(Int.1)
            r.num * (r.num * r.num) = -(Int.1)
            r.num * (r.num * r.num) = Int.2
            -(Int.1) = Int.2
            Int.2 = -(Int.1)
            Int.2 = Int.from_nat(Nat.2)
            -(Int.1) = -(Int.from_nat(Nat.1))
            Int.from_nat(Nat.2) = -(Int.from_nat(Nat.1))
            from_eq_neg_from(Nat.2, Nat.1)
            Nat.2 = Nat.0
            false
        }
        if r.num = Int.2 {
            r.num * (r.num * r.num) = Int.2 * (Int.2 * Int.2)
            Int.2 = Int.from_nat(Nat.2)
            mul_nat_from_nat_right(Int.from_nat(Nat.2), Nat.2)
            Int.from_nat(Nat.2).mul_nat(Nat.2) = Int.from_nat(Nat.2) * Int.from_nat(Nat.2)
            mul_nat_from_nat_left(Nat.2, Nat.2)
            Int.from_nat(Nat.2).mul_nat(Nat.2) = Int.from_nat(Nat.2 * Nat.2)
            Nat.2 * Nat.2 = Nat.4
            Int.from_nat(Nat.2) * Int.from_nat(Nat.2) = Int.from_nat(Nat.4)
            Int.2 * Int.2 = Int.4
            Int.2 * (Int.2 * Int.2) = Int.2 * Int.4
            Int.2 = Int.from_nat(Nat.2)
            Int.4 = Int.from_nat(Nat.4)
            Int.8 = Int.from_nat(Nat.8)
            Int.2 * Int.4 = Int.from_nat(Nat.2) * Int.from_nat(Nat.4)
            mul_nat_from_nat_right(Int.from_nat(Nat.2), Nat.4)
            Int.from_nat(Nat.2).mul_nat(Nat.4) = Int.from_nat(Nat.2) * Int.from_nat(Nat.4)
            mul_nat_from_nat_left(Nat.2, Nat.4)
            Int.from_nat(Nat.2).mul_nat(Nat.4) = Int.from_nat(Nat.2 * Nat.4)
            Nat.2 * Nat.4 = Nat.8
            Int.from_nat(Nat.2) * Int.from_nat(Nat.4) = Int.from_nat(Nat.8)
            Int.2 * Int.4 = Int.8
            r.num * (r.num * r.num) = Int.8
            r.num * (r.num * r.num) = Int.2
            Int.8 = Int.2
            Int.8 = Int.from_nat(Nat.8)
            Int.2 = Int.from_nat(Nat.2)
            Int.from_nat(Nat.8) = Int.from_nat(Nat.2)
            pos_part_from(Nat.8)
            Int.from_nat(Nat.8).pos_part = Nat.8
            pos_part_from(Nat.2)
            Int.from_nat(Nat.2).pos_part = Nat.2
            Nat.8 = Nat.2
            lt_suc(Nat.2)
            Nat.2 < Nat.3
            lt_suc(Nat.3)
            Nat.3 < Nat.4
            lt_suc(Nat.4)
            Nat.4 < Nat.5
            lt_suc(Nat.5)
            Nat.5 < Nat.6
            lt_suc(Nat.6)
            Nat.6 < Nat.7
            lt_suc(Nat.7)
            Nat.7 < Nat.8
            lt_trans(Nat.2, Nat.3, Nat.4)
            Nat.2 < Nat.4
            lt_trans(Nat.2, Nat.4, Nat.5)
            Nat.2 < Nat.5
            lt_trans(Nat.2, Nat.5, Nat.6)
            Nat.2 < Nat.6
            lt_trans(Nat.2, Nat.6, Nat.7)
            Nat.2 < Nat.7
            lt_trans(Nat.2, Nat.7, Nat.8)
            Nat.2 < Nat.8
            Nat.2 < Nat.2
            lt_not_ref(Nat.2)
            not Nat.2 < Nat.2
            false
        }
        if r.num = -(Int.2) {
            r.num * (r.num * r.num) = (-(Int.2)) * ((-(Int.2)) * (-(Int.2)))
            mul_neg_neg(Int.2, Int.2)
            (-(Int.2)) * (-(Int.2)) = Int.2 * Int.2
            Int.2 = Int.from_nat(Nat.2)
            mul_nat_from_nat_right(Int.from_nat(Nat.2), Nat.2)
            Int.from_nat(Nat.2).mul_nat(Nat.2) = Int.from_nat(Nat.2) * Int.from_nat(Nat.2)
            mul_nat_from_nat_left(Nat.2, Nat.2)
            Int.from_nat(Nat.2).mul_nat(Nat.2) = Int.from_nat(Nat.2 * Nat.2)
            Nat.2 * Nat.2 = Nat.4
            Int.from_nat(Nat.2) * Int.from_nat(Nat.2) = Int.from_nat(Nat.4)
            Int.2 * Int.2 = Int.4
            (-(Int.2)) * ((-(Int.2)) * (-(Int.2))) = (-(Int.2)) * Int.4
            (-(Int.2)) * Int.4 = -(Int.2 * Int.4)
            Int.2 = Int.from_nat(Nat.2)
            Int.4 = Int.from_nat(Nat.4)
            Int.2 * Int.4 = Int.from_nat(Nat.2) * Int.from_nat(Nat.4)
            mul_nat_from_nat_right(Int.from_nat(Nat.2), Nat.4)
            Int.from_nat(Nat.2).mul_nat(Nat.4) = Int.from_nat(Nat.2) * Int.from_nat(Nat.4)
            mul_nat_from_nat_left(Nat.2, Nat.4)
            Int.from_nat(Nat.2).mul_nat(Nat.4) = Int.from_nat(Nat.2 * Nat.4)
            Nat.2 * Nat.4 = Nat.8
            Int.from_nat(Nat.2) * Int.from_nat(Nat.4) = Int.from_nat(Nat.8)
            Int.2 * Int.4 = Int.8
            (-(Int.2)) * Int.4 = -(Int.8)
            r.num * (r.num * r.num) = -(Int.8)
            r.num * (r.num * r.num) = Int.2
            -(Int.8) = Int.2
            Int.2 = -(Int.8)
            Int.2 = Int.from_nat(Nat.2)
            -(Int.8) = -(Int.from_nat(Nat.8))
            Int.from_nat(Nat.2) = -(Int.from_nat(Nat.8))
            from_eq_neg_from(Nat.2, Nat.8)
            Nat.2 = Nat.0
            false
        }
        r.num != Int.1
        r.num != -(Int.1)
        r.num != Int.2
        r.num != -(Int.2)
        false
    }
}
