/// Fermat's theorem on sums of two squares: an odd prime is a sum of two
/// squares exactly when it is congruent to one modulo four.
///
/// The forward direction follows the classical route: for p ≡ 1 (mod 4), the
/// second supplement to quadratic reciprocity supplies a square root of minus
/// one modulo p, and a pigeonhole (Thue) argument over a square grid of side
/// about sqrt(p) converts that congruence into an exact representation
/// p = a*a + b*b with 0 < a*a + b*b < 2p.
from nat import Nat, add_mod, div_mod_decomp, mod_lt, small_mod, mod_of_zero,
    div_imp_mod, divides_self, mul_to_zero, strong_induction, lt_or_lte,
    lte_trans, lte_and_lt, lt_imp_lte_suc, add_imp_sub, add_imp_sub_left,
    sub_pos, add_one_right, lt_suc, lt_suc_right, not_lt_zero,
    alt_suc_ne_zero, lte_mul, lte_imp_not_lt, sub_zero, exp_zero, sq_eq_mul,
    add_cancels_left, add_cancels_right, lte_mul_both, lte_mul_left,
    lte_mul_right, divides_lte, divides_add, divides_sub, divides_mul,
    divides_trans, lt_mul_both, lt_cancel_mul, lte_add_left, lte_add_right,
    pos_of_ne_zero, mul_cancel_left, mul_to_one, lt_imp_lt_suc, lte_antisymm,
    add_sub, sub_self, sub_lt, two_divides_suc_iff, divides_zero, zero_divides,
    lte_cancel_suc, mod_by_zero, add_to_zero
from number_theory.congruence import congr_mod_refl, congr_mod_symm,
    congr_mod_trans, congr_mod_add, congr_mod_mul,
    mod_congr_mod_self, congr_mod_zero_iff_divides, divides_of_congr_mod_zero,
    congr_mod_zero_of_divides, mod_add_eq, mod_mul_eq, mod_add_mul
from number_theory.quadratic_residue import is_quadratic_residue_mod
from number_theory.four_squares import square_mul_square
from number_theory.totient import congr_mod_add_cancel_right_pos,
    congr_mod_add_cancel_right_zero
from number_theory.quadratic_residue_supplements import prime_even_half_pred_is_quadratic_residue
from list import List, map, length_range, range_is_unique, range_contains_of_lt,
    range_contains_iff_lt, lt_of_range_contains
from data.finite.finite_fiber_partition import finite_list_pigeonhole_into_shorter_list
numerals Nat

// ============================================================================
// Section 1: a prime congruent to one modulo four has a square root of minus
// one modulo itself.
// ============================================================================

/// A prime congruent to one modulo four has a square root of minus one
/// modulo itself. This is the second supplement to quadratic reciprocity,
/// expressed here through the library's half-exponent theorems.
theorem prime_sqrt_neg_one_congr(p: Nat) {
    p.is_prime and p.mod(Nat.4) = Nat.1 implies
        exists(x: Nat) { x.pow(Nat.2).congr_mod(p - Nat.1, p) }
} by {
    if p.is_prime and p.mod(Nat.4) = Nat.1 {
        add_mod(p, Nat.4)
        let q: Nat satisfy { q * Nat.4 + p.mod(Nat.4) = p }
        p.mod(Nat.4) = Nat.1
        q * Nat.4 + Nat.1 = p
        q * Nat.4 = Nat.4 * q
        Nat.4 * q + Nat.1 = p
        let h: Nat = Nat.2 * q
        Nat.2 * h = Nat.4 * q
        Nat.2 * h + Nat.1 = p
        if h = Nat.0 {
            Nat.2 * h = Nat.0
            p = Nat.1
            Nat.1 < p
            false
        }
        h != Nat.0
        Nat.2 * q = h
        exists(c: Nat) { Nat.2 * c = h }
        Nat.2.divides(h)
        prime_even_half_pred_is_quadratic_residue(p, h)
        is_quadratic_residue_mod(p - Nat.1, p)
        is_quadratic_residue_mod(p - Nat.1, p) =
            exists(x: Nat) { x.pow(Nat.2).congr_mod(p - Nat.1, p) }
        exists(x: Nat) { x.pow(Nat.2).congr_mod(p - Nat.1, p) }
    }
}

/// A prime congruent to one modulo four divides one plus a square.
theorem prime_divides_sq_plus_one(p: Nat) {
    p.is_prime and p.mod(Nat.4) = Nat.1 implies exists(x: Nat) {
        p.divides(x * x + Nat.1)
    }
} by {
    if p.is_prime and p.mod(Nat.4) = Nat.1 {
        prime_sqrt_neg_one_congr(p)
        let x: Nat satisfy { x.pow(Nat.2).congr_mod(p - Nat.1, p) }
        sq_eq_mul(x)
        x.pow(Nat.2) = x * x
        congr_mod_symm(x.pow(Nat.2), p - Nat.1, p)
        (p - Nat.1).congr_mod(x.pow(Nat.2), p)
        (p - Nat.1).congr_mod(x * x, p)
        congr_mod_add(p - Nat.1, x * x, Nat.1, Nat.1, p)
        (p - Nat.1 + Nat.1).congr_mod(x * x + Nat.1, p)
        Nat.1 < p
        lt_imp_lte_suc(Nat.1, p)
        Nat.1 <= p
        add_sub(p, Nat.1)
        p - Nat.1 + Nat.1 = p
        p.congr_mod(x * x + Nat.1, p)
        divides_self(p)
        div_imp_mod(p, p)
        p.mod(p) = Nat.0
        mod_of_zero(p)
        Nat.0.mod(p) = Nat.0
        p.congr_mod(Nat.0, p)
        congr_mod_symm(p, x * x + Nat.1, p)
        (x * x + Nat.1).congr_mod(p, p)
        congr_mod_trans(x * x + Nat.1, p, Nat.0, p)
        (x * x + Nat.1).congr_mod(Nat.0, p)
        congr_mod_zero_iff_divides(p, x * x + Nat.1)
        p.divides(x * x + Nat.1)
        exists(y: Nat) { p.divides(y * y + Nat.1) }
    }
}

// ============================================================================
// Section 2: the integer square root of a prime.
// ============================================================================

/// A number at most `b` and distinct from `b` is strictly below `b`.
theorem lte_neq_imp_lt(a: Nat, b: Nat) {
    a <= b and a != b implies a < b
} by {
    if a <= b and a != b {
        lt_or_lte(a, b)
        if b <= a {
            lte_antisymm(a, b)
            a = b
            false
        }
        a < b
    }
}

/// A number not at most `b` is strictly above `b`.
theorem not_lte_imp_gt(a: Nat, b: Nat) {
    not (a <= b) implies b < a
} by {
    if not (a <= b) {
        lt_or_lte(b, a)
        if a <= b {
            false
        }
        b < a
    }
}

/// Scans downward from `fuel`, returning the largest `k <= fuel` whose square
/// is at most `p` (zero when no positive square fits).
define isqrt_fuel(p: Nat, fuel: Nat) -> Nat {
    match fuel {
        Nat.zero {
            Nat.0
        }
        Nat.suc(f) {
            if f.suc * f.suc <= p {
                f.suc
            } else {
                isqrt_fuel(p, f)
            }
        }
    }
}

/// The integer square root of `p`, computed with the full fuel `p`.
define isqrt(p: Nat) -> Nat {
    isqrt_fuel(p, p)
}

/// The result of the downward scan is at most the fuel.
theorem isqrt_fuel_lte_fuel(p: Nat, fuel: Nat) {
    isqrt_fuel(p, fuel) <= fuel
} by {
    define pred(k: Nat) -> Bool {
        isqrt_fuel(p, k) <= k
    }
    isqrt_fuel(p, Nat.0) = Nat.0
    pred(Nat.0)
    forall(f: Nat) {
        if pred(f) {
            if f.suc * f.suc <= p {
                isqrt_fuel(p, f.suc) = f.suc
                f.suc <= f.suc
                pred(f.suc)
            } else {
                isqrt_fuel(p, f.suc) = isqrt_fuel(p, f)
                pred(f)
                isqrt_fuel(p, f) <= f
                f <= f.suc
                lte_trans(isqrt_fuel(p, f), f, f.suc)
                isqrt_fuel(p, f) <= f.suc
                pred(f.suc)
            }
        }
    }
    pred(fuel)
}

/// The square of the downward-scan result is at most `p`.
theorem isqrt_fuel_sq_le(p: Nat, fuel: Nat) {
    isqrt_fuel(p, fuel) * isqrt_fuel(p, fuel) <= p
} by {
    define pred(k: Nat) -> Bool {
        isqrt_fuel(p, k) * isqrt_fuel(p, k) <= p
    }
    isqrt_fuel(p, Nat.0) = Nat.0
    isqrt_fuel(p, Nat.0) * isqrt_fuel(p, Nat.0) = Nat.0
    Nat.0 <= p
    isqrt_fuel(p, Nat.0) * isqrt_fuel(p, Nat.0) <= p
    pred(Nat.0)
    forall(f: Nat) {
        if pred(f) {
            if f.suc * f.suc <= p {
                isqrt_fuel(p, f.suc) = f.suc
                f.suc * f.suc <= p
                pred(f.suc)
            } else {
                isqrt_fuel(p, f.suc) = isqrt_fuel(p, f)
                pred(f)
                isqrt_fuel(p, f) * isqrt_fuel(p, f) <= p
                pred(f.suc)
            }
        }
    }
    pred(fuel)
}

/// Every number strictly between the scan result and the fuel has square
/// strictly above `p`.
theorem isqrt_fuel_above(p: Nat, fuel: Nat, j: Nat) {
    isqrt_fuel(p, fuel) < j and j <= fuel implies p < j * j
} by {
    define pred(k: Nat) -> Bool {
        forall(y: Nat) {
            isqrt_fuel(p, k) < y and y <= k implies p < y * y
        }
    }
    isqrt_fuel(p, Nat.0) = Nat.0
    pred(Nat.0)
    forall(f: Nat) {
        if pred(f) {
            forall(y: Nat) {
                if isqrt_fuel(p, f.suc) < y and y <= f.suc {
                    if f.suc * f.suc <= p {
                        isqrt_fuel(p, f.suc) = f.suc
                        f.suc < y
                        y <= f.suc
                        lte_imp_not_lt(f.suc, y)
                        false
                    } else {
                        isqrt_fuel(p, f.suc) = isqrt_fuel(p, f)
                        isqrt_fuel(p, f) < y
                        if y = f.suc {
                            y * y = f.suc * f.suc
                            not_lte_imp_gt(f.suc * f.suc, p)
                            p < f.suc * f.suc
                            p < y * y
                        } else {
                            y != f.suc
                            lte_neq_imp_lt(y, f.suc)
                            y < f.suc
                            lt_imp_lte_suc(y, f.suc)
                            y.suc <= f.suc
                            lte_cancel_suc(y, f)
                            y <= f
                            pred(f)
                            (isqrt_fuel(p, f) < y and y <= f implies p < y * y)
                            p < y * y
                        }
                        p < y * y
                    }
                    p < y * y
                }
            }
            pred(f.suc)
        }
    }
    pred(fuel)
}

/// The integer square root is at most `p`.
theorem isqrt_lte_self(p: Nat) {
    isqrt(p) <= p
} by {
    isqrt_fuel_lte_fuel(p, p)
    isqrt(p) = isqrt_fuel(p, p)
    isqrt(p) <= p
}

/// The square of the integer square root is at most `p`.
theorem isqrt_sq_le(p: Nat) {
    isqrt(p) * isqrt(p) <= p
} by {
    isqrt_fuel_sq_le(p, p)
    isqrt(p) = isqrt_fuel(p, p)
    isqrt(p) * isqrt(p) <= p
}

/// A prime is not a perfect square, so the square of its integer square root
/// is strictly below it.
theorem isqrt_sq_lt_prime(p: Nat) {
    p.is_prime implies isqrt(p) * isqrt(p) < p
} by {
    if p.is_prime {
        isqrt_sq_le(p)
        isqrt(p) * isqrt(p) <= p
        if isqrt(p) * isqrt(p) = p {
            exists(c: Nat) { isqrt(p) * c = p }
            isqrt(p).divides(p)
            if isqrt(p) = Nat.0 {
                isqrt(p) * isqrt(p) = Nat.0
                p = Nat.0
                Nat.1 < p
                Nat.1 < Nat.0
                false
            }
            if isqrt(p) = Nat.1 {
                isqrt(p) * isqrt(p) = Nat.1
                p = Nat.1
                Nat.1 < p
                false
            }
            isqrt(p) != Nat.0
            isqrt(p) != Nat.1
            if Nat.1 < isqrt(p) {
                exists(b: Nat, c: Nat) {
                    Nat.1 < b and Nat.1 < c and p = b * c
                }
                p.is_composite
                p.is_prime = Nat.1 < p and not p.is_composite
                not p.is_composite
                false
            }
            lt_or_lte(isqrt(p), Nat.1)
            if Nat.1 <= isqrt(p) {
                lte_neq_imp_lt(Nat.1, isqrt(p))
                Nat.1 < isqrt(p)
                false
            }
            isqrt(p) < Nat.1
            lt_suc_right(isqrt(p), Nat.0)
            if isqrt(p) = Nat.0 {
                false
            }
            not_lt_zero(isqrt(p))
            false
        }
        isqrt(p) * isqrt(p) != p
        lte_neq_imp_lt(isqrt(p) * isqrt(p), p)
        isqrt(p) * isqrt(p) < p
    }
}

/// The successor of the integer square root of a prime has square strictly
/// above the prime.
theorem isqrt_suc_sq_gt(p: Nat) {
    p.is_prime implies p < (isqrt(p) + Nat.1) * (isqrt(p) + Nat.1)
} by {
    if p.is_prime {
        isqrt_lte_self(p)
        isqrt(p) <= p
        if p <= isqrt(p) {
            lte_antisymm(isqrt(p), p)
            isqrt(p) = p
            isqrt_sq_le(p)
            isqrt(p) * isqrt(p) <= p
            p * p <= p
            Nat.1 < p
            pos_of_ne_zero(p)
            p != Nat.0
            lt_mul_both(p, Nat.1, p)
            p * Nat.1 < p * p
            p < p * p
            lte_imp_not_lt(p * p, p)
            false
        }
        isqrt(p) < p
        lt_imp_lte_suc(isqrt(p), p)
        isqrt(p).suc <= p
        isqrt(p) + Nat.1 = isqrt(p).suc
        isqrt(p) + Nat.1 <= p
        isqrt_fuel_above(p, p, isqrt(p) + Nat.1)
        (isqrt_fuel(p, p) < isqrt(p) + Nat.1 and isqrt(p) + Nat.1 <= p
            implies p < (isqrt(p) + Nat.1) * (isqrt(p) + Nat.1))
        isqrt(p) = isqrt_fuel(p, p)
        lt_suc(isqrt(p))
        isqrt(p) < isqrt(p) + Nat.1
        isqrt_fuel(p, p) < isqrt(p) + Nat.1
        p < (isqrt(p) + Nat.1) * (isqrt(p) + Nat.1)
    }
}

// ============================================================================
// Section 3: modular arithmetic lemmas for the Thue argument.
// ============================================================================

/// Squaring is monotone on natural numbers.
theorem lte_sq(a: Nat, b: Nat) {
    a <= b implies a * a <= b * b
} by {
    if a <= b {
        lte_mul_both(a, a, b)
        a * a <= a * b
        lte_mul_right(b, a, b)
        a * b <= b * b
        lte_trans(a * a, a * b, b * b)
        a * a <= b * b
    }
}

/// Adding the same natural to both sides of a congruence cancels.
theorem congr_mod_add_cancel_left(x: Nat, u: Nat, v: Nat, n: Nat) {
    (x + u).congr_mod(x + v, n) implies u.congr_mod(v, n)
} by {
    if (x + u).congr_mod(x + v, n) {
        if n = Nat.0 {
            x + u = u + x
            x + v = v + x
            (u + x).congr_mod(v + x, Nat.0)
            congr_mod_add_cancel_right_zero(u, v, x)
            u.congr_mod(v, Nat.0)
            u.congr_mod(v, n)
        } else {
            n != Nat.0
            x + u = u + x
            x + v = v + x
            (u + x).congr_mod(v + x, n)
            congr_mod_add_cancel_right_pos(u, v, x, n)
            u.congr_mod(v, n)
        }
    }
}

/// A sum congruent to zero has congruent squares: `(u + v) ≡ 0` implies
/// `u * u ≡ v * v`. Both `(u + v) * u` and `(u + v) * v` are congruent to
/// zero, and after expanding they share the cross term `u * v`, which
/// cancels.
theorem congr_mod_sq_of_sum_zero(u: Nat, v: Nat, n: Nat) {
    (u + v).congr_mod(Nat.0, n) implies (u * u).congr_mod(v * v, n)
} by {
    if (u + v).congr_mod(Nat.0, n) {
        if n = Nat.0 {
            mod_by_zero(u + v)
            mod_by_zero(Nat.0)
            (u + v).mod(Nat.0) = u + v
            Nat.0.mod(Nat.0) = Nat.0
            (u + v).congr_mod(Nat.0, Nat.0) = ((u + v).mod(Nat.0) = Nat.0.mod(Nat.0))
            u + v = Nat.0
            add_to_zero(u, v)
            u = Nat.0 and v = Nat.0
            u * u = Nat.0
            v * v = Nat.0
            congr_mod_refl(Nat.0, Nat.0)
            Nat.0.congr_mod(Nat.0, Nat.0)
            (u * u).congr_mod(v * v, Nat.0)
            (u * u).congr_mod(v * v, n)
        } else {
            congr_mod_mul(u + v, Nat.0, u, u, n)
            ((u + v) * u).congr_mod(Nat.0 * u, n)
            Nat.0 * u = Nat.0
            ((u + v) * u).congr_mod(Nat.0, n)
            congr_mod_mul(u + v, Nat.0, v, v, n)
            ((u + v) * v).congr_mod(Nat.0 * v, n)
            Nat.0 * v = Nat.0
            ((u + v) * v).congr_mod(Nat.0, n)
            (u + v) * u = u * u + v * u
            (u + v) * v = u * v + v * v
            (u * u + v * u).congr_mod(Nat.0, n)
            (u * v + v * v).congr_mod(Nat.0, n)
            v * u = u * v
            u * u + v * u = u * u + u * v
            (u * u + u * v).congr_mod(Nat.0, n)
            u * v + v * v = v * v + u * v
            (v * v + u * v).congr_mod(Nat.0, n)
            congr_mod_symm(v * v + u * v, Nat.0, n)
            Nat.0.congr_mod(v * v + u * v, n)
            congr_mod_trans(u * u + u * v, Nat.0, v * v + u * v, n)
            (u * u + u * v).congr_mod(v * v + u * v, n)
            congr_mod_add_cancel_left(u * v, u * u, v * v, n)
            (u * u).congr_mod(v * v, n)
        }
    }
}

// ============================================================================
// Section 4: the Thue pigeonhole argument.
// ============================================================================

/// The residue of the pair-encoded index `i`: the index `i` encodes the pair
/// `(i.div(m+1), i.mod(m+1))`, and this maps it to `(a + b * x) mod p`.
define thue_residue(p: Nat, m: Nat, x: Nat, i: Nat) -> Nat {
    (i.div(m + Nat.1) + (i.mod(m + Nat.1)) * x).mod(p)
}

/// The partial application of `thue_residue` for use with `map`.
define thue_residue_fn(p: Nat, m: Nat, x: Nat) -> (Nat -> Nat) {
    function(i: Nat) { thue_residue(p, m, x, i) }
}

/// The quotient of an index below a square of `m + 1` is at most `m`.
theorem div_lt_square_le(m: Nat, i: Nat) {
    i < (m + Nat.1) * (m + Nat.1) implies i.div(m + Nat.1) <= m
} by {
    if i < (m + Nat.1) * (m + Nat.1) {
        div_mod_decomp(i, m + Nat.1)
        i.div(m + Nat.1) * (m + Nat.1) + i.mod(m + Nat.1) = i
        i.div(m + Nat.1) * (m + Nat.1) <= i
        lte_and_lt(i.div(m + Nat.1) * (m + Nat.1), i, (m + Nat.1) * (m + Nat.1))
        i.div(m + Nat.1) * (m + Nat.1) < (m + Nat.1) * (m + Nat.1)
        i.div(m + Nat.1) * (m + Nat.1) = (m + Nat.1) * i.div(m + Nat.1)
        (m + Nat.1) * i.div(m + Nat.1) < (m + Nat.1) * (m + Nat.1)
        alt_suc_ne_zero(m)
        m + Nat.1 != Nat.0
        lt_cancel_mul(m + Nat.1, i.div(m + Nat.1), m + Nat.1)
        i.div(m + Nat.1) < m + Nat.1
        lt_imp_lte_suc(i.div(m + Nat.1), m + Nat.1)
        i.div(m + Nat.1).suc <= m + Nat.1
        lte_cancel_suc(i.div(m + Nat.1), m)
        i.div(m + Nat.1) <= m
    }
}

/// The remainder of an index below a square of `m + 1` is at most `m`.
theorem mod_lt_square_le(m: Nat, i: Nat) {
    i < (m + Nat.1) * (m + Nat.1) implies i.mod(m + Nat.1) <= m
} by {
    if i < (m + Nat.1) * (m + Nat.1) {
        alt_suc_ne_zero(m)
        m + Nat.1 != Nat.0
        mod_lt(i, m + Nat.1)
        i.mod(m + Nat.1) < m + Nat.1
        lt_imp_lte_suc(i.mod(m + Nat.1), m + Nat.1)
        i.mod(m + Nat.1).suc <= m + Nat.1
        lte_cancel_suc(i.mod(m + Nat.1), m)
        i.mod(m + Nat.1) <= m
    }
}

/// The pigeonhole step: more than `p` encoded indices below a square of side
/// `m + 1` map into the `p` residues, so two distinct indices collide.
theorem thue_collision(p: Nat, m: Nat, x: Nat) {
    p != Nat.0 and m * m < p and p < (m + Nat.1) * (m + Nat.1) implies
        exists(i: Nat, j: Nat) {
            i < (m + Nat.1) * (m + Nat.1) and j < (m + Nat.1) * (m + Nat.1) and
            i != j and thue_residue(p, m, x, i) = thue_residue(p, m, x, j)
        }
} by {
    if p != Nat.0 and m * m < p and p < (m + Nat.1) * (m + Nat.1) {
        finite_list_pigeonhole_into_shorter_list[Nat, Nat](
            ((m + Nat.1) * (m + Nat.1)).range, p.range, thue_residue_fn(p, m, x))
        (((m + Nat.1) * (m + Nat.1)).range.is_unique and
            ((m + Nat.1) * (m + Nat.1)).range.length > p.range.length and
            forall(z: Nat) {
                ((m + Nat.1) * (m + Nat.1)).range.contains(z) implies
                    p.range.contains(thue_residue_fn(p, m, x)(z))
            }) implies exists(z1: Nat, z2: Nat) {
                ((m + Nat.1) * (m + Nat.1)).range.contains(z1) and
                ((m + Nat.1) * (m + Nat.1)).range.contains(z2) and
                z1 != z2 and thue_residue_fn(p, m, x)(z1) = thue_residue_fn(p, m, x)(z2)
            }
        range_is_unique((m + Nat.1) * (m + Nat.1))
        ((m + Nat.1) * (m + Nat.1)).range.is_unique
        length_range((m + Nat.1) * (m + Nat.1))
        ((m + Nat.1) * (m + Nat.1)).range.length = (m + Nat.1) * (m + Nat.1)
        length_range(p)
        p.range.length = p
        p < (m + Nat.1) * (m + Nat.1)
        p.range.length < ((m + Nat.1) * (m + Nat.1)).range.length
        ((m + Nat.1) * (m + Nat.1)).range.length > p.range.length
        forall(z: Nat) {
            if ((m + Nat.1) * (m + Nat.1)).range.contains(z) {
                lt_of_range_contains((m + Nat.1) * (m + Nat.1), z)
                z < (m + Nat.1) * (m + Nat.1)
                thue_residue_fn(p, m, x)(z) = thue_residue(p, m, x, z)
                p != Nat.0
                mod_lt(z.div(m + Nat.1) + (z.mod(m + Nat.1)) * x, p)
                thue_residue(p, m, x, z) < p
                thue_residue_fn(p, m, x)(z) < p
                range_contains_of_lt(p, thue_residue_fn(p, m, x)(z))
                p.range.contains(thue_residue_fn(p, m, x)(z))
            }
        }
        exists(z1: Nat, z2: Nat) {
            ((m + Nat.1) * (m + Nat.1)).range.contains(z1) and
            ((m + Nat.1) * (m + Nat.1)).range.contains(z2) and
            z1 != z2 and thue_residue_fn(p, m, x)(z1) = thue_residue_fn(p, m, x)(z2)
        }
        let (z1: Nat, z2: Nat) satisfy {
            ((m + Nat.1) * (m + Nat.1)).range.contains(z1) and
            ((m + Nat.1) * (m + Nat.1)).range.contains(z2) and
            z1 != z2 and thue_residue_fn(p, m, x)(z1) = thue_residue_fn(p, m, x)(z2)
        }
        lt_of_range_contains((m + Nat.1) * (m + Nat.1), z1)
        lt_of_range_contains((m + Nat.1) * (m + Nat.1), z2)
        thue_residue_fn(p, m, x)(z1) = thue_residue(p, m, x, z1)
        thue_residue_fn(p, m, x)(z2) = thue_residue(p, m, x, z2)
        thue_residue(p, m, x, z1) = thue_residue(p, m, x, z2)
        exists(i: Nat, j: Nat) {
            i < (m + Nat.1) * (m + Nat.1) and j < (m + Nat.1) * (m + Nat.1) and
            i != j and thue_residue(p, m, x, i) = thue_residue(p, m, x, j)
        }
    }
}

/// A positive multiple of `p` strictly below `2p` is `p` itself.
theorem sum_of_squares_eq_p(p: Nat, c: Nat, d: Nat) {
    p != Nat.0 and p.divides(c * c + d * d) and c * c + d * d != Nat.0 and
        c * c + d * d < Nat.2 * p implies c * c + d * d = p
} by {
    if p != Nat.0 and p.divides(c * c + d * d) and c * c + d * d != Nat.0 and
            c * c + d * d < Nat.2 * p {
        p.divides(c * c + d * d) = exists(q: Nat) { p * q = c * c + d * d }
        let q: Nat satisfy { p * q = c * c + d * d }
        if q = Nat.0 {
            p * q = Nat.0
            c * c + d * d = Nat.0
            false
        }
        q != Nat.0
        pos_of_ne_zero(q)
        Nat.0 < q
        c * c + d * d = p * q
        p * q < Nat.2 * p
        Nat.2 * p = p * Nat.2
        p * q < p * Nat.2
        lt_cancel_mul(p, q, Nat.2)
        q < Nat.2
        lt_suc_right(q, Nat.1)
        if q < Nat.1 {
            lt_suc_right(q, Nat.0)
            if q = Nat.0 {
                false
            }
            not_lt_zero(q)
            false
        }
        q = Nat.1
        p * q = p
        c * c + d * d = p
    }
}

/// From a collision `(a + b*x) ≡ (a' + b'*x) (mod p)` the pair of absolute
/// differences `c = |a - a'|`, `d = |b - b'|` satisfies `c*c ≡ d*d*x*x
/// (mod p)`. The four cases (each coordinate of the first pair dominating or
/// not) all reduce to cancelling a common offset in a congruence.
theorem thue_sq_congr(a: Nat, b: Nat, x: Nat, a2: Nat, b2: Nat, p: Nat) {
    (a + b * x).congr_mod(a2 + b2 * x, p) implies
        exists(c: Nat, d: Nat) {
            (a = a2 + c or a2 = a + c) and (b = b2 + d or b2 = b + d) and
            (c * c).congr_mod((d * d) * (x * x), p)
        }
} by {
    if (a + b * x).congr_mod(a2 + b2 * x, p) {
        if a2 <= a {
            if b2 <= b {
                add_sub(a, a2)
                (a - a2) + a2 = a
                a2 + (a - a2) = (a - a2) + a2
                a = a2 + (a - a2)
                add_sub(b, b2)
                (b - b2) + b2 = b
                b2 + (b - b2) = (b - b2) + b2
                b = b2 + (b - b2)
                b * x = (b2 + (b - b2)) * x
                (b2 + (b - b2)) * x = b2 * x + (b - b2) * x
                b * x = b2 * x + (b - b2) * x
                a + b * x = (a2 + (a - a2)) + (b2 * x + (b - b2) * x)
                (a2 + (a - a2)) + (b2 * x + (b - b2) * x) =
                    a2 + b2 * x + ((a - a2) + (b - b2) * x)
                a + b * x = (a2 + b2 * x) + ((a - a2) + (b - b2) * x)
                ((a2 + b2 * x) + ((a - a2) + (b - b2) * x)).congr_mod(a2 + b2 * x, p)
                congr_mod_add_cancel_left(a2 + b2 * x,
                    (a - a2) + (b - b2) * x, Nat.0, p)
                ((a - a2) + (b - b2) * x).congr_mod(Nat.0, p)
                congr_mod_sq_of_sum_zero(a - a2, (b - b2) * x, p)
                ((a - a2) * (a - a2)).congr_mod(
                    ((b - b2) * x) * ((b - b2) * x), p)
                square_mul_square(b - b2, x)
                ((b - b2) * x) * ((b - b2) * x) = (b - b2) * (b - b2) * (x * x)
                ((a - a2) * (a - a2)).congr_mod(
                    ((b - b2) * (b - b2)) * (x * x), p)
                exists(c: Nat, d: Nat) {
                    (a = a2 + c or a2 = a + c) and (b = b2 + d or b2 = b + d) and
                    (c * c).congr_mod((d * d) * (x * x), p)
                }
            } else {
                add_sub(a, a2)
                (a - a2) + a2 = a
                a2 + (a - a2) = (a - a2) + a2
                a = a2 + (a - a2)
                add_sub(b2, b)
                (b2 - b) + b = b2
                b + (b2 - b) = (b2 - b) + b
                b2 = b + (b2 - b)
                a + b * x = (a2 + b * x) + (a - a2)
                b2 * x = (b + (b2 - b)) * x
                (b + (b2 - b)) * x = b * x + (b2 - b) * x
                b2 * x = b * x + (b2 - b) * x
                a2 + b2 * x = a2 + (b * x + (b2 - b) * x)
                a2 + (b * x + (b2 - b) * x) = (a2 + b * x) + (b2 - b) * x
                a2 + b2 * x = (a2 + b * x) + (b2 - b) * x
                ((a2 + b * x) + (a - a2)).congr_mod(
                    (a2 + b * x) + (b2 - b) * x, p)
                congr_mod_add_cancel_left(a2 + b * x,
                    a - a2, (b2 - b) * x, p)
                (a - a2).congr_mod((b2 - b) * x, p)
                congr_mod_mul(a - a2, (b2 - b) * x, a - a2, (b2 - b) * x, p)
                ((a - a2) * (a - a2)).congr_mod(
                    ((b2 - b) * x) * ((b2 - b) * x), p)
                square_mul_square(b2 - b, x)
                ((b2 - b) * x) * ((b2 - b) * x) = (b2 - b) * (b2 - b) * (x * x)
                ((a - a2) * (a - a2)).congr_mod(
                    ((b2 - b) * (b2 - b)) * (x * x), p)
                exists(c: Nat, d: Nat) {
                    (a = a2 + c or a2 = a + c) and (b = b2 + d or b2 = b + d) and
                    (c * c).congr_mod((d * d) * (x * x), p)
                }
            }
            exists(c: Nat, d: Nat) {
                (a = a2 + c or a2 = a + c) and (b = b2 + d or b2 = b + d) and
                (c * c).congr_mod((d * d) * (x * x), p)
            }
        } else {
            lt_or_lte(a, a2)
            if a2 <= a {
                false
            }
            a < a2
            if b2 <= b {
                add_sub(a2, a)
                (a2 - a) + a = a2
                a + (a2 - a) = (a2 - a) + a
                a2 = a + (a2 - a)
                add_sub(b, b2)
                (b - b2) + b2 = b
                b2 + (b - b2) = (b - b2) + b2
                b = b2 + (b - b2)
                b * x = (b2 + (b - b2)) * x
                (b2 + (b - b2)) * x = b2 * x + (b - b2) * x
                b * x = b2 * x + (b - b2) * x
                a + b * x = a + (b2 * x + (b - b2) * x)
                a + (b2 * x + (b - b2) * x) = (a + b2 * x) + (b - b2) * x
                a + b * x = (a + b2 * x) + (b - b2) * x
                a2 + b2 * x = (a + b2 * x) + (a2 - a)
                ((a + b2 * x) + (b - b2) * x).congr_mod(
                    (a + b2 * x) + (a2 - a), p)
                congr_mod_add_cancel_left(a + b2 * x,
                    (b - b2) * x, a2 - a, p)
                ((b - b2) * x).congr_mod(a2 - a, p)
                congr_mod_symm((b - b2) * x, a2 - a, p)
                (a2 - a).congr_mod((b - b2) * x, p)
                congr_mod_mul(a2 - a, (b - b2) * x, a2 - a, (b - b2) * x, p)
                ((a2 - a) * (a2 - a)).congr_mod(
                    ((b - b2) * x) * ((b - b2) * x), p)
                square_mul_square(b - b2, x)
                ((b - b2) * x) * ((b - b2) * x) = (b - b2) * (b - b2) * (x * x)
                ((a2 - a) * (a2 - a)).congr_mod(
                    ((b - b2) * (b - b2)) * (x * x), p)
                exists(c: Nat, d: Nat) {
                    (a = a2 + c or a2 = a + c) and (b = b2 + d or b2 = b + d) and
                    (c * c).congr_mod((d * d) * (x * x), p)
                }
            } else {
                add_sub(a2, a)
                (a2 - a) + a = a2
                a + (a2 - a) = (a2 - a) + a
                a2 = a + (a2 - a)
                add_sub(b2, b)
                (b2 - b) + b = b2
                b + (b2 - b) = (b2 - b) + b
                b2 = b + (b2 - b)
                b2 * x = (b + (b2 - b)) * x
                (b + (b2 - b)) * x = b * x + (b2 - b) * x
                b2 * x = b * x + (b2 - b) * x
                a2 + b2 * x = (a + (a2 - a)) + (b * x + (b2 - b) * x)
                (a + (a2 - a)) + (b * x + (b2 - b) * x) =
                    a + b * x + ((a2 - a) + (b2 - b) * x)
                a2 + b2 * x = a + b * x + ((a2 - a) + (b2 - b) * x)
                ((a + b * x) + ((a2 - a) + (b2 - b) * x)).congr_mod(a + b * x, p)
                congr_mod_add_cancel_left(a + b * x,
                    (a2 - a) + (b2 - b) * x, Nat.0, p)
                ((a2 - a) + (b2 - b) * x).congr_mod(Nat.0, p)
                congr_mod_sq_of_sum_zero(a2 - a, (b2 - b) * x, p)
                ((a2 - a) * (a2 - a)).congr_mod(
                    ((b2 - b) * x) * ((b2 - b) * x), p)
                square_mul_square(b2 - b, x)
                ((b2 - b) * x) * ((b2 - b) * x) = (b2 - b) * (b2 - b) * (x * x)
                ((a2 - a) * (a2 - a)).congr_mod(
                    ((b2 - b) * (b2 - b)) * (x * x), p)
                exists(c: Nat, d: Nat) {
                    (a = a2 + c or a2 = a + c) and (b = b2 + d or b2 = b + d) and
                    (c * c).congr_mod((d * d) * (x * x), p)
                }
            }
            exists(c: Nat, d: Nat) {
                (a = a2 + c or a2 = a + c) and (b = b2 + d or b2 = b + d) and
                (c * c).congr_mod((d * d) * (x * x), p)
            }
        }
        exists(c: Nat, d: Nat) {
            (a = a2 + c or a2 = a + c) and (b = b2 + d or b2 = b + d) and
            (c * c).congr_mod((d * d) * (x * x), p)
        }
    }
}

/// The Thue assembly: a square root of minus one modulo `p` with a grid
/// bound `m*m < p < (m+1)*(m+1)` yields an exact representation `p = a*a + b*b`.
theorem thue_sum_of_squares(p: Nat, m: Nat, x: Nat) {
    p != Nat.0 and m * m < p and p < (m + Nat.1) * (m + Nat.1) and
        x.pow(Nat.2).congr_mod(p - Nat.1, p) implies
        exists(a: Nat, b: Nat) { a * a + b * b = p }
} by {
    if p != Nat.0 and m * m < p and p < (m + Nat.1) * (m + Nat.1) and
            x.pow(Nat.2).congr_mod(p - Nat.1, p) {
        thue_collision(p, m, x)
        let (i: Nat, j: Nat) satisfy {
            i < (m + Nat.1) * (m + Nat.1) and j < (m + Nat.1) * (m + Nat.1) and
            i != j and thue_residue(p, m, x, i) = thue_residue(p, m, x, j)
        }
        let a: Nat = i.div(m + Nat.1)
        let b: Nat = i.mod(m + Nat.1)
        let a2: Nat = j.div(m + Nat.1)
        let b2: Nat = j.mod(m + Nat.1)
        thue_residue(p, m, x, i) = (i.div(m + Nat.1) + (i.mod(m + Nat.1)) * x).mod(p)
        thue_residue(p, m, x, j) = (j.div(m + Nat.1) + (j.mod(m + Nat.1)) * x).mod(p)
        (i.div(m + Nat.1) + (i.mod(m + Nat.1)) * x).mod(p) =
            (j.div(m + Nat.1) + (j.mod(m + Nat.1)) * x).mod(p)
        (a + b * x).mod(p) = (a2 + b2 * x).mod(p)
        (a + b * x).congr_mod(a2 + b2 * x, p)
        thue_sq_congr(a, b, x, a2, b2, p)
        let (c: Nat, d: Nat) satisfy {
            (a = a2 + c or a2 = a + c) and (b = b2 + d or b2 = b + d) and
            (c * c).congr_mod((d * d) * (x * x), p)
        }
        sq_eq_mul(x)
        x.pow(Nat.2) = x * x
        (x * x).congr_mod(p - Nat.1, p)
        congr_mod_mul(x * x, p - Nat.1, d * d, d * d, p)
        ((x * x) * (d * d)).congr_mod((p - Nat.1) * (d * d), p)
        (x * x) * (d * d) = (d * d) * (x * x)
        ((d * d) * (x * x)).congr_mod((p - Nat.1) * (d * d), p)
        (p - Nat.1) * (d * d) = (d * d) * (p - Nat.1)
        ((d * d) * (x * x)).congr_mod((d * d) * (p - Nat.1), p)
        congr_mod_trans(c * c, (d * d) * (x * x), (d * d) * (p - Nat.1), p)
        (c * c).congr_mod((d * d) * (p - Nat.1), p)
        (d * d) * (p - Nat.1) + d * d = (d * d) * p
        exists(k: Nat) { p * k = (d * d) * p }
        p.divides((d * d) * p)
        congr_mod_zero_of_divides(p, (d * d) * p)
        ((d * d) * p).congr_mod(Nat.0, p)
        ((d * d) * (p - Nat.1) + d * d).congr_mod(Nat.0, p)
        congr_mod_add(c * c, (d * d) * (p - Nat.1), d * d, d * d, p)
        (c * c + d * d).congr_mod((d * d) * (p - Nat.1) + d * d, p)
        congr_mod_trans(c * c + d * d, (d * d) * (p - Nat.1) + d * d, Nat.0, p)
        (c * c + d * d).congr_mod(Nat.0, p)
        divides_of_congr_mod_zero(p, c * c + d * d)
        p.divides(c * c + d * d)
        div_lt_square_le(m, i)
        a <= m
        div_lt_square_le(m, j)
        a2 <= m
        mod_lt_square_le(m, i)
        b <= m
        mod_lt_square_le(m, j)
        b2 <= m
        if a = a2 + c {
            a2 + c = a
            c <= a2 + c
            c <= a
            lte_trans(c, a, m)
            c <= m
        } else {
            a2 = a + c
            a + c = a2
            c <= a + c
            c <= a2
            lte_trans(c, a2, m)
            c <= m
        }
        if b = b2 + d {
            b2 + d = b
            d <= b2 + d
            d <= b
            lte_trans(d, b, m)
            d <= m
        } else {
            b2 = b + d
            b + d = b2
            d <= b + d
            d <= b2
            lte_trans(d, b2, m)
            d <= m
        }
        lte_sq(c, m)
        c * c <= m * m
        lte_sq(d, m)
        d * d <= m * m
        lte_add_left(d * d, c * c, m * m)
        d * d + c * c <= d * d + m * m
        lte_add_right(m * m, d * d, m * m)
        d * d + m * m <= m * m + m * m
        lte_trans(d * d + c * c, d * d + m * m, m * m + m * m)
        d * d + c * c <= m * m + m * m
        d * d + c * c = c * c + d * d
        m * m + m * m = Nat.2 * (m * m)
        c * c + d * d <= Nat.2 * (m * m)
        alt_suc_ne_zero(Nat.1)
        Nat.2 != Nat.0
        lt_mul_both(Nat.2, m * m, p)
        Nat.2 * (m * m) < Nat.2 * p
        lte_and_lt(c * c + d * d, Nat.2 * (m * m), Nat.2 * p)
        c * c + d * d < Nat.2 * p
        if c * c + d * d = Nat.0 {
            add_to_zero(c * c, d * d)
            c * c = Nat.0 and d * d = Nat.0
            c * c = Nat.0
            mul_to_zero(c, c)
            c = Nat.0
            d * d = Nat.0
            mul_to_zero(d, d)
            d = Nat.0
            if a = a2 + c {
                c = Nat.0
                a = a2 + Nat.0
                a = a2
            } else {
                a2 = a + c
                c = Nat.0
                a2 = a + Nat.0
                a2 = a
                a = a2
            }
            if b = b2 + d {
                d = Nat.0
                b = b2 + Nat.0
                b = b2
            } else {
                b2 = b + d
                d = Nat.0
                b2 = b + Nat.0
                b2 = b
                b = b2
            }
            div_mod_decomp(i, m + Nat.1)
            a * (m + Nat.1) + b = i
            div_mod_decomp(j, m + Nat.1)
            a2 * (m + Nat.1) + b2 = j
            a2 * (m + Nat.1) + b2 = a * (m + Nat.1) + b
            i = j
            false
        }
        c * c + d * d != Nat.0
        sum_of_squares_eq_p(p, c, d)
        c * c + d * d = p
        exists(u: Nat, v: Nat) { u * u + v * v = p }
    }
}

/// An odd prime congruent to one modulo four is a sum of two squares.
theorem prime_sum_of_two_squares(p: Nat) {
    p.is_prime and p.mod(Nat.4) = Nat.1 implies exists(a: Nat, b: Nat) {
        a * a + b * b = p
    }
} by {
    if p.is_prime and p.mod(Nat.4) = Nat.1 {
        isqrt_sq_lt_prime(p)
        isqrt(p) * isqrt(p) < p
        isqrt_suc_sq_gt(p)
        p < (isqrt(p) + Nat.1) * (isqrt(p) + Nat.1)
        prime_sqrt_neg_one_congr(p)
        let x: Nat satisfy { x.pow(Nat.2).congr_mod(p - Nat.1, p) }
        Nat.1 < p
        pos_of_ne_zero(p)
        p != Nat.0
        thue_sum_of_squares(p, isqrt(p), x)
        exists(a: Nat, b: Nat) { a * a + b * b = p }
    }
}

/// Two is a sum of two squares.
theorem two_is_sum_of_two_squares {
    exists(a: Nat, b: Nat) { a * a + b * b = Nat.2 }
} by {
    Nat.1 * Nat.1 + Nat.1 * Nat.1 = Nat.2
    exists(a: Nat, b: Nat) { a * a + b * b = Nat.2 }
}

// ============================================================================
// Section 5: the converse — a sum of two squares that is an odd prime is
// congruent to one modulo four.
// ============================================================================

/// One modulo four is one.
theorem one_mod_four {
    Nat.1.mod(Nat.4) = Nat.1
} by {
    lt_suc(Nat.1)
    Nat.1 < Nat.2
    lt_imp_lt_suc(Nat.1, Nat.2)
    Nat.1 < Nat.3
    lt_imp_lt_suc(Nat.1, Nat.3)
    Nat.1 < Nat.4
    small_mod(Nat.1, Nat.4)
    Nat.1.mod(Nat.4) = Nat.1
}

/// Two modulo four is two.
theorem two_mod_four {
    Nat.2.mod(Nat.4) = Nat.2
} by {
    lt_suc(Nat.2)
    Nat.2 < Nat.3
    lt_imp_lt_suc(Nat.2, Nat.3)
    Nat.2 < Nat.4
    small_mod(Nat.2, Nat.4)
    Nat.2.mod(Nat.4) = Nat.2
}

/// The square of a natural number is congruent to zero or one modulo four.
theorem square_mod_four_case(a: Nat) {
    (a * a).mod(Nat.4) = Nat.0 or (a * a).mod(Nat.4) = Nat.1
} by {
    let r: Nat = a.mod(Nat.4)
    add_mod(a, Nat.4)
    let q: Nat satisfy { q * Nat.4 + a.mod(Nat.4) = a }
    q * Nat.4 + r = a
    mod_congr_mod_self(a, Nat.4)
    a.mod(Nat.4).congr_mod(a, Nat.4)
    congr_mod_symm(a.mod(Nat.4), a, Nat.4)
    a.congr_mod(a.mod(Nat.4), Nat.4)
    a.congr_mod(r, Nat.4)
    congr_mod_mul(a, r, a, r, Nat.4)
    (a * a).congr_mod(r * r, Nat.4)
    (a * a).mod(Nat.4) = (r * r).mod(Nat.4)
    alt_suc_ne_zero(Nat.3)
    Nat.4 != Nat.0
    mod_lt(a, Nat.4)
    r < Nat.4
    lt_suc_right(r, Nat.3)
    if r != Nat.3 {
        r < Nat.3
        lt_suc_right(r, Nat.2)
        if r != Nat.2 {
            r < Nat.2
            lt_suc_right(r, Nat.1)
            if r != Nat.1 {
                r < Nat.1
                lt_suc_right(r, Nat.0)
                if r != Nat.0 {
                    r < Nat.0
                    not_lt_zero(r)
                    false
                }
                r = Nat.0
                r * r = Nat.0
                mod_of_zero(Nat.4)
                Nat.0.mod(Nat.4) = Nat.0
                (r * r).mod(Nat.4) = Nat.0
                (a * a).mod(Nat.4) = Nat.0
                (a * a).mod(Nat.4) = Nat.0 or (a * a).mod(Nat.4) = Nat.1
            } else {
                r = Nat.1
                r * r = Nat.1
                one_mod_four
                Nat.1.mod(Nat.4) = Nat.1
                (r * r).mod(Nat.4) = Nat.1
                (a * a).mod(Nat.4) = Nat.1
                (a * a).mod(Nat.4) = Nat.0 or (a * a).mod(Nat.4) = Nat.1
            }
        } else {
            r = Nat.2
            r * r = Nat.4
            divides_self(Nat.4)
            div_imp_mod(Nat.4, Nat.4)
            Nat.4.mod(Nat.4) = Nat.0
            (r * r).mod(Nat.4) = Nat.0
            (a * a).mod(Nat.4) = Nat.0
            (a * a).mod(Nat.4) = Nat.0 or (a * a).mod(Nat.4) = Nat.1
        }
    } else {
        r = Nat.3
        r * r = Nat.9
        Nat.9 = Nat.2 * Nat.4 + Nat.1
        mod_add_mul(Nat.2, Nat.4, Nat.1)
        (Nat.2 * Nat.4 + Nat.1).mod(Nat.4) = Nat.1.mod(Nat.4)
        small_mod(Nat.1, Nat.4)
        Nat.1.mod(Nat.4) = Nat.1
        Nat.9.mod(Nat.4) = Nat.1
        (r * r).mod(Nat.4) = Nat.1
        (a * a).mod(Nat.4) = Nat.1
        (a * a).mod(Nat.4) = Nat.0 or (a * a).mod(Nat.4) = Nat.1
    }
}

/// The sum of two squares is congruent to zero, one, or two modulo four.
theorem sum_two_squares_mod_four_case(a: Nat, b: Nat) {
    (a * a + b * b).mod(Nat.4) = Nat.0 or
        (a * a + b * b).mod(Nat.4) = Nat.1 or
        (a * a + b * b).mod(Nat.4) = Nat.2
} by {
    square_mod_four_case(a)
    (a * a).mod(Nat.4) = Nat.0 or (a * a).mod(Nat.4) = Nat.1
    square_mod_four_case(b)
    (b * b).mod(Nat.4) = Nat.0 or (b * b).mod(Nat.4) = Nat.1
    mod_add_eq(a * a, b * b, Nat.4)
    (a * a + b * b).mod(Nat.4) =
        ((a * a).mod(Nat.4) + (b * b).mod(Nat.4)).mod(Nat.4)
    if (a * a).mod(Nat.4) != Nat.0 {
        (a * a).mod(Nat.4) = Nat.1
        if (b * b).mod(Nat.4) != Nat.0 {
            (b * b).mod(Nat.4) = Nat.1
            (a * a).mod(Nat.4) + (b * b).mod(Nat.4) = Nat.2
            ((a * a).mod(Nat.4) + (b * b).mod(Nat.4)).mod(Nat.4) = Nat.2.mod(Nat.4)
            two_mod_four
            Nat.2.mod(Nat.4) = Nat.2
            ((a * a).mod(Nat.4) + (b * b).mod(Nat.4)).mod(Nat.4) = Nat.2
            (a * a + b * b).mod(Nat.4) = Nat.2
            (a * a + b * b).mod(Nat.4) = Nat.0 or
                (a * a + b * b).mod(Nat.4) = Nat.1 or
                (a * a + b * b).mod(Nat.4) = Nat.2
        } else {
            (b * b).mod(Nat.4) = Nat.0
            (a * a).mod(Nat.4) + (b * b).mod(Nat.4) = Nat.1
            ((a * a).mod(Nat.4) + (b * b).mod(Nat.4)).mod(Nat.4) = Nat.1.mod(Nat.4)
            small_mod(Nat.1, Nat.4)
            Nat.1.mod(Nat.4) = Nat.1
            ((a * a).mod(Nat.4) + (b * b).mod(Nat.4)).mod(Nat.4) = Nat.1
            (a * a + b * b).mod(Nat.4) = Nat.1
            (a * a + b * b).mod(Nat.4) = Nat.0 or
                (a * a + b * b).mod(Nat.4) = Nat.1 or
                (a * a + b * b).mod(Nat.4) = Nat.2
        }
        (a * a + b * b).mod(Nat.4) = Nat.0 or
            (a * a + b * b).mod(Nat.4) = Nat.1 or
            (a * a + b * b).mod(Nat.4) = Nat.2
    } else {
        (a * a).mod(Nat.4) = Nat.0
        if (b * b).mod(Nat.4) != Nat.0 {
            (b * b).mod(Nat.4) = Nat.1
            (a * a).mod(Nat.4) + (b * b).mod(Nat.4) = Nat.1
            ((a * a).mod(Nat.4) + (b * b).mod(Nat.4)).mod(Nat.4) = Nat.1.mod(Nat.4)
            small_mod(Nat.1, Nat.4)
            Nat.1.mod(Nat.4) = Nat.1
            ((a * a).mod(Nat.4) + (b * b).mod(Nat.4)).mod(Nat.4) = Nat.1
            (a * a + b * b).mod(Nat.4) = Nat.1
            (a * a + b * b).mod(Nat.4) = Nat.0 or
                (a * a + b * b).mod(Nat.4) = Nat.1 or
                (a * a + b * b).mod(Nat.4) = Nat.2
        } else {
            (b * b).mod(Nat.4) = Nat.0
            (a * a).mod(Nat.4) + (b * b).mod(Nat.4) = Nat.0
            ((a * a).mod(Nat.4) + (b * b).mod(Nat.4)).mod(Nat.4) = Nat.0.mod(Nat.4)
            mod_of_zero(Nat.4)
            Nat.0.mod(Nat.4) = Nat.0
            ((a * a).mod(Nat.4) + (b * b).mod(Nat.4)).mod(Nat.4) = Nat.0
            (a * a + b * b).mod(Nat.4) = Nat.0
            (a * a + b * b).mod(Nat.4) = Nat.0 or
                (a * a + b * b).mod(Nat.4) = Nat.1 or
                (a * a + b * b).mod(Nat.4) = Nat.2
        }
        (a * a + b * b).mod(Nat.4) = Nat.0 or
            (a * a + b * b).mod(Nat.4) = Nat.1 or
            (a * a + b * b).mod(Nat.4) = Nat.2
    }
}

/// `4q + 2` factors as `2 * (2q + 1)`.
theorem mul_four_add_two(q: Nat) {
    q * Nat.4 + Nat.2 = Nat.2 * (Nat.2 * q + Nat.1)
} by {
    q * Nat.4 = Nat.4 * q
    Nat.4 * q = Nat.2 * (Nat.2 * q)
    q * Nat.4 = Nat.2 * (Nat.2 * q)
    q * Nat.4 + Nat.2 = Nat.2 * (Nat.2 * q) + Nat.2
    Nat.2 * (Nat.2 * q) + Nat.2 = Nat.2 * (Nat.2 * q) + Nat.2 * Nat.1
    Nat.2 * (Nat.2 * q) + Nat.2 * Nat.1 = Nat.2 * (Nat.2 * q + Nat.1)
    q * Nat.4 + Nat.2 = Nat.2 * (Nat.2 * q + Nat.1)
}

/// A prime divisible by two is two.
theorem prime_two_divides_imp_eq_two(p: Nat) {
    p.is_prime and Nat.2.divides(p) implies p = Nat.2
} by {
    if p.is_prime and Nat.2.divides(p) {
        Nat.2.divides(p) = exists(c: Nat) { Nat.2 * c = p }
        let c: Nat satisfy { Nat.2 * c = p }
        if c = Nat.0 {
            Nat.2 * c = Nat.0
            p = Nat.0
            false
        }
        c != Nat.0
        if Nat.1 < c {
            exists(b: Nat, c2: Nat) { Nat.1 < b and Nat.1 < c2 and p = b * c2 }
            p.is_composite
            p.is_prime = Nat.1 < p and not p.is_composite
            not p.is_composite
            false
        }
        lt_or_lte(Nat.1, c)
        if Nat.1 < c {
            false
        }
        c <= Nat.1
        lt_suc_right(c, Nat.1)
        if c < Nat.1 {
            lt_suc_right(c, Nat.0)
            if c = Nat.0 {
                false
            }
            not_lt_zero(c)
            false
        }
        c = Nat.1
        Nat.2 * c = Nat.2
        p = Nat.2
    }
}

/// A prime that is a sum of two squares and is not two is congruent to one
/// modulo four.
theorem prime_sum_two_squares_converse(p: Nat) {
    p.is_prime and p != Nat.2 and exists(a: Nat, b: Nat) { a * a + b * b = p }
        implies p.mod(Nat.4) = Nat.1
} by {
    if p.is_prime and p != Nat.2 and exists(a: Nat, b: Nat) { a * a + b * b = p } {
        let (a: Nat, b: Nat) satisfy { a * a + b * b = p }
        sum_two_squares_mod_four_case(a, b)
        (a * a + b * b).mod(Nat.4) = Nat.0 or
            (a * a + b * b).mod(Nat.4) = Nat.1 or
            (a * a + b * b).mod(Nat.4) = Nat.2
        a * a + b * b = p
        p.mod(Nat.4) = (a * a + b * b).mod(Nat.4)
        if (a * a + b * b).mod(Nat.4) != Nat.0 {
            if (a * a + b * b).mod(Nat.4) != Nat.2 {
                (a * a + b * b).mod(Nat.4) = Nat.1
                p.mod(Nat.4) = Nat.1
                p.mod(Nat.4) = Nat.1
            } else {
                (a * a + b * b).mod(Nat.4) = Nat.2
                p.mod(Nat.4) = Nat.2
                add_mod(p, Nat.4)
                let q: Nat satisfy { q * Nat.4 + p.mod(Nat.4) = p }
                q * Nat.4 + Nat.2 = p
                p = q * Nat.4 + Nat.2
                mul_four_add_two(q)
                q * Nat.4 + Nat.2 = Nat.2 * (Nat.2 * q + Nat.1)
                Nat.2 * (Nat.2 * q + Nat.1) = p
                exists(c: Nat) { Nat.2 * c = p }
                Nat.2.divides(p)
                prime_two_divides_imp_eq_two(p)
                p = Nat.2
                false
            }
            p.mod(Nat.4) = Nat.1
        } else {
            (a * a + b * b).mod(Nat.4) = Nat.0
            p.mod(Nat.4) = Nat.0
            add_mod(p, Nat.4)
            let q: Nat satisfy { q * Nat.4 + p.mod(Nat.4) = p }
            q * Nat.4 + Nat.0 = p
            q * Nat.4 = p
            q * Nat.4 = Nat.2 * (Nat.2 * q)
            Nat.2 * (Nat.2 * q) = p
            exists(c: Nat) { Nat.2 * c = p }
            Nat.2.divides(p)
            prime_two_divides_imp_eq_two(p)
            p = Nat.2
            false
        }
        p.mod(Nat.4) = Nat.1
    }
}