from int import Int, abs, neg_or_pos, abs_mul, mul_from_nat, add_from_nat, abs_from_nat,
    mul_neg_neg, mul_comm, mul_assoc, mul_distrib_right, mul_distrib_left,
    mul_sub_distrib_right, mul_sub_distrib_left, add_comm, add_assoc, add_comm_4, add_zero_left,
    add_zero_right, add_neg, neg_distrib, neg_neg, mul_neg_left, mul_neg_right, mul_one_right,
    mul_one_left, sub_zero_right, sub_self, neg_sub, sub_add_cancel_left, sub_add_cancel_right,
    is_unit, two_units, abs_decomp, div_from_nat, div_imp_div_abs, div_abs_imp_div,
    euclids_lemma_prime, is_prime, division_theorem,
    exp_add, exp_mul, sq_eq_mul, lte_nonnegative_ints_implies_nats, add_right_cancel,
    lt_from_nat, nonzero_pos_or_neg,
    pos_is_not_neg, one_pos, zero_not_neg, add_lte, neg_lt_zero, nonpos_lt_pos, from_nat_pos,
    lte_abs, lt_mul_pos, abs_neg, mul_pos_neg
from algebra.add import Add
from algebra.mul import Mul
from algebra.neg import Neg
from algebra.zero import Zero
from algebra.one import One
from nat import Nat, add_cancels_left, lte_trans, lt_imp_lte_suc, lte_mul_both, lt_mul_both,
    pos_of_ne_zero, lte_mul, mul_cancel_left, mul_cancel_right, distrib_left, distrib_right,
    add_sub, sub_lt, sub_pos, add_imp_sub, div_mod_decomp, mod_lt, lt_or_lte, lte_antisymm,
    not_lt_zero, lt_suc, lt_suc_right, lt_trans, lte_add_left, lte_add_right, lt_not_ref,
    divides_symm, divides_self, divides_mul, gcd_of_prime, lte_and_lt
from number_theory.fermat import prime_divides_mul
from number_theory.sum_of_two_squares import prime_sum_of_two_squares, prime_sum_two_squares_converse
from number_theory.infinite_descent import two_squares_descent
from number_theory.arithmetic_functions import nat_mul_eq_one
numerals Int

/// The Gaussian integers: numbers of the form a + bi with integer real and
/// imaginary parts, where i² = -1.  Every Gaussian integer is represented by
/// its real part `re` and imaginary part `im`.
structure GaussInt {
    /// The real part of the Gaussian integer.
    re: Int
    /// The imaginary part of the Gaussian integer.
    im: Int
}

/// The Gaussian integer zero.
instance GaussInt: Zero {
    let 0: GaussInt = GaussInt.new(Int.0, Int.0)
}

/// The Gaussian integer one.
instance GaussInt: One {
    let 1: GaussInt = GaussInt.new(Int.1, Int.0)
}

/// The sum of two Gaussian integers is computed componentwise.
instance GaussInt: Add {
    define add(self, other: GaussInt) -> GaussInt {
        GaussInt.new(self.re + other.re, self.im + other.im)
    }
}

/// The negation of a Gaussian integer negates both components.
instance GaussInt: Neg {
    define neg(self) -> GaussInt {
        GaussInt.new(-self.re, -self.im)
    }
}

/// The product of two Gaussian integers follows the rule
/// (a + bi)(c + di) = (ac - bd) + (ad + bc)i.
instance GaussInt: Mul {
    define mul(self, other: GaussInt) -> GaussInt {
        GaussInt.new(
            self.re * other.re - self.im * other.im,
            self.re * other.im + self.im * other.re
        )
    }
}

attributes GaussInt {
    /// The imaginary unit i = 0 + 1i, satisfying i² = -1.
    let i: GaussInt = GaussInt.new(Int.0, Int.1)

    /// The complex conjugate, which negates the imaginary part.
    define conj(self) -> GaussInt {
        GaussInt.new(self.re, -self.im)
    }

    /// The norm N(a + bi) = a² + b², an integer (always nonnegative in
    /// practice).  The norm is multiplicative and controls divisibility.
    define norm(self) -> Int {
        self.re * self.re + self.im * self.im
    }

    /// The norm expressed as a natural number, N(a + bi) = |a|² + |b|².
    define norm_nat(self) -> Nat {
        abs(self.re) * abs(self.re) + abs(self.im) * abs(self.im)
    }

    /// True when the Gaussian integer divides another: z | w when w = z·k
    /// for some Gaussian integer k.
    define divides(self, other: GaussInt) -> Bool {
        exists(k: GaussInt) { self * k = other }
    }

    /// True when the Gaussian integer is a unit, i.e. has a multiplicative
    /// inverse.  The units of the Gaussian integers are ±1 and ±i.
    define is_unit(self) -> Bool {
        exists(k: GaussInt) { self * k = GaussInt.1 }
    }
}

// ============================================================================
// Section 1: ring laws of the Gaussian integers
// ============================================================================

/// A difference of a sum is a chain of differences.
theorem int_sub_sum(x: Int, y: Int, z: Int) {
    x - (y + z) = x - y - z
} by {
    x - (y + z) = x + -(y + z)
    neg_distrib(y, z)
    -(y + z) = -y + -z
    x + (-y + -z) = (x + -y) + -z
    (x + -y) + -z = x - y - z
    x - (y + z) = x - y - z
}

/// A chain of differences is invariant under cyclically moving the first
/// subtracted term to the end.
theorem int_sub_chain_swap(x: Int, p: Int, q: Int, r: Int) {
    x - p - q - r = x - q - r - p
} by {
    x - p - q - r = ((x + -p) + -q) + -r
    ((x + -p) + -q) + -r = x + (-p + (-q + -r))
    -p + (-q + -r) = -q + (-r + -p)
    x + (-q + (-r + -p)) = ((x + -q) + -r) + -p
    ((x + -q) + -r) + -p = x - q - r - p
    x - p - q - r = x - q - r - p
}

/// A negated term may move to the end of a sum.
theorem int_add_sub_reorder(x: Int, y: Int, z: Int, w: Int) {
    x - w + y + z = x + y + z - w
} by {
    x - w + y + z = x + -w + y + z
    x + -w + y + z = x + y + z + -w
    x + y + z + -w = x + y + z - w
    x - w + y + z = x + y + z - w
}

/// Two negated terms may be moved to the end of a sum.
theorem int_sub_sub_reorder(x: Int, y: Int, z: Int, w: Int) {
    x - y + z - w = x + z - y - w
} by {
    x - y + z - w = x + -y + z + -w
    x + -y + z + -w = x + z + -y + -w
    x + z + -y + -w = x + z - y - w
    x - y + z - w = x + z - y - w
}

/// A difference trailing a sum may be regrouped as a difference of sums.
theorem int_add_sub_assoc_right(x: Int, y: Int, z: Int, w: Int) {
    x + y + (z - w) = x + y + z - w
} by {
    x + y + (z - w) = x + y + (z + -w)
    x + y + (z + -w) = (x + y + z) + -w
    (x + y + z) + -w = x + y + z - w
    x + y + (z - w) = x + y + z - w
}

/// Addition of Gaussian integers is commutative.
theorem gauss_add_comm(a: GaussInt, b: GaussInt) {
    a + b = b + a
} by {
    (a + b).re = a.re + b.re
    (a + b).im = a.im + b.im
    (b + a).re = b.re + a.re
    (b + a).im = b.im + a.im
    a.re + b.re = b.re + a.re
    a.im + b.im = b.im + a.im
}

/// Addition of Gaussian integers is associative.
theorem gauss_add_assoc(a: GaussInt, b: GaussInt, c: GaussInt) {
    (a + b) + c = a + (b + c)
} by {
    ((a + b) + c).re = (a + b).re + c.re
    (a + b).re = a.re + b.re
    ((a + b) + c).re = a.re + b.re + c.re
    (a + (b + c)).re = a.re + (b + c).re
    (b + c).re = b.re + c.re
    (a + (b + c)).re = a.re + (b.re + c.re)
    a.re + (b.re + c.re) = a.re + b.re + c.re
    ((a + b) + c).re = (a + (b + c)).re
    ((a + b) + c).im = (a + b).im + c.im
    (a + b).im = a.im + b.im
    ((a + b) + c).im = a.im + b.im + c.im
    (a + (b + c)).im = a.im + (b + c).im
    (b + c).im = b.im + c.im
    (a + (b + c)).im = a.im + (b.im + c.im)
    a.im + (b.im + c.im) = a.im + b.im + c.im
    ((a + b) + c).im = (a + (b + c)).im
}

/// Zero is the additive identity on the left.
theorem gauss_add_zero_left(a: GaussInt) {
    GaussInt.0 + a = a
} by {
    (GaussInt.0 + a).re = GaussInt.0.re + a.re
    GaussInt.0.re = Int.0
    (GaussInt.0 + a).re = Int.0 + a.re
    Int.0 + a.re = a.re
    (GaussInt.0 + a).re = a.re
    (GaussInt.0 + a).im = GaussInt.0.im + a.im
    GaussInt.0.im = Int.0
    (GaussInt.0 + a).im = Int.0 + a.im
    Int.0 + a.im = a.im
    (GaussInt.0 + a).im = a.im
}

/// Zero is the additive identity on the right.
theorem gauss_add_zero_right(a: GaussInt) {
    a + GaussInt.0 = a
} by {
    (a + GaussInt.0).re = a.re + GaussInt.0.re
    GaussInt.0.re = Int.0
    (a + GaussInt.0).re = a.re + Int.0
    a.re + Int.0 = a.re
    (a + GaussInt.0).re = a.re
    (a + GaussInt.0).im = a.im + GaussInt.0.im
    GaussInt.0.im = Int.0
    (a + GaussInt.0).im = a.im + Int.0
    a.im + Int.0 = a.im
    (a + GaussInt.0).im = a.im
}

/// Adding a Gaussian integer and its negation gives zero.
theorem gauss_add_neg(a: GaussInt) {
    a + -a = GaussInt.0
} by {
    (a + -a).re = a.re + (-a).re
    (-a).re = -(a.re)
    (a + -a).re = a.re + -(a.re)
    a.re + -(a.re) = Int.0
    (a + -a).re = Int.0
    GaussInt.0.re = Int.0
    (a + -a).re = GaussInt.0.re
    (a + -a).im = a.im + (-a).im
    (-a).im = -(a.im)
    (a + -a).im = a.im + -(a.im)
    a.im + -(a.im) = Int.0
    (a + -a).im = Int.0
    GaussInt.0.im = Int.0
    (a + -a).im = GaussInt.0.im
}

/// Multiplication of Gaussian integers is commutative.
theorem gauss_mul_comm(a: GaussInt, b: GaussInt) {
    a * b = b * a
} by {
    (a * b).re = a.re * b.re - a.im * b.im
    (b * a).re = b.re * a.re - b.im * a.im
    a.re * b.re = b.re * a.re
    a.im * b.im = b.im * a.im
    (a * b).re = (b * a).re
    (a * b).im = a.re * b.im + a.im * b.re
    (b * a).im = b.re * a.im + b.im * a.re
    a.re * b.im = b.im * a.re
    a.im * b.re = b.re * a.im
    (a * b).im = (b * a).im
}

/// Multiplication of Gaussian integers is associative.
theorem gauss_mul_assoc(a: GaussInt, b: GaussInt, c: GaussInt) {
    (a * b) * c = a * (b * c)
} by {
    (a * b).re = a.re * b.re - a.im * b.im
    (a * b).im = a.re * b.im + a.im * b.re
    ((a * b) * c).re = (a * b).re * c.re - (a * b).im * c.im
    ((a * b) * c).re = (a.re * b.re - a.im * b.im) * c.re -
        (a.re * b.im + a.im * b.re) * c.im
    (a.re * b.re - a.im * b.im) * c.re = a.re * b.re * c.re - a.im * b.im * c.re
    (a.re * b.im + a.im * b.re) * c.im = a.re * b.im * c.im + a.im * b.re * c.im
    ((a * b) * c).re = a.re * b.re * c.re - a.im * b.im * c.re -
        (a.re * b.im * c.im + a.im * b.re * c.im)
    int_sub_sum(a.re * b.re * c.re - a.im * b.im * c.re, a.re * b.im * c.im,
        a.im * b.re * c.im)
    a.re * b.re * c.re - a.im * b.im * c.re - (a.re * b.im * c.im + a.im * b.re * c.im) =
        a.re * b.re * c.re - a.im * b.im * c.re - a.re * b.im * c.im - a.im * b.re * c.im
    ((a * b) * c).re = a.re * b.re * c.re - a.im * b.im * c.re -
        a.re * b.im * c.im - a.im * b.re * c.im
    (b * c).re = b.re * c.re - b.im * c.im
    (b * c).im = b.re * c.im + b.im * c.re
    (a * (b * c)).re = a.re * (b * c).re - a.im * (b * c).im
    (a * (b * c)).re = a.re * (b.re * c.re - b.im * c.im) -
        a.im * (b.re * c.im + b.im * c.re)
    a.re * (b.re * c.re - b.im * c.im) = a.re * b.re * c.re - a.re * b.im * c.im
    a.im * (b.re * c.im + b.im * c.re) = a.im * b.re * c.im + a.im * b.im * c.re
    (a * (b * c)).re = a.re * b.re * c.re - a.re * b.im * c.im -
        (a.im * b.re * c.im + a.im * b.im * c.re)
    int_sub_sum(a.re * b.re * c.re - a.re * b.im * c.im, a.im * b.re * c.im,
        a.im * b.im * c.re)
    a.re * b.re * c.re - a.re * b.im * c.im - (a.im * b.re * c.im + a.im * b.im * c.re) =
        a.re * b.re * c.re - a.re * b.im * c.im - a.im * b.re * c.im - a.im * b.im * c.re
    (a * (b * c)).re = a.re * b.re * c.re - a.re * b.im * c.im -
        a.im * b.re * c.im - a.im * b.im * c.re
    int_sub_chain_swap(a.re * b.re * c.re, a.im * b.im * c.re, a.re * b.im * c.im,
        a.im * b.re * c.im)
    a.re * b.re * c.re - a.im * b.im * c.re - a.re * b.im * c.im - a.im * b.re * c.im =
        a.re * b.re * c.re - a.re * b.im * c.im - a.im * b.re * c.im - a.im * b.im * c.re
    ((a * b) * c).re = (a * (b * c)).re
    ((a * b) * c).im = (a * b).re * c.im + (a * b).im * c.re
    ((a * b) * c).im = (a.re * b.re - a.im * b.im) * c.im +
        (a.re * b.im + a.im * b.re) * c.re
    (a.re * b.re - a.im * b.im) * c.im = a.re * b.re * c.im - a.im * b.im * c.im
    (a.re * b.im + a.im * b.re) * c.re = a.re * b.im * c.re + a.im * b.re * c.re
    ((a * b) * c).im = a.re * b.re * c.im - a.im * b.im * c.im +
        a.re * b.im * c.re + a.im * b.re * c.re
    (a * (b * c)).im = a.re * (b * c).im + a.im * (b * c).re
    (a * (b * c)).im = a.re * (b.re * c.im + b.im * c.re) +
        a.im * (b.re * c.re - b.im * c.im)
    a.re * (b.re * c.im + b.im * c.re) = a.re * b.re * c.im + a.re * b.im * c.re
    a.im * (b.re * c.re - b.im * c.im) = a.im * b.re * c.re - a.im * b.im * c.im
    (a * (b * c)).im = a.re * b.re * c.im + a.re * b.im * c.re +
        (a.im * b.re * c.re - a.im * b.im * c.im)
    int_add_sub_assoc_right(a.re * b.re * c.im, a.re * b.im * c.re, a.im * b.re * c.re,
        a.im * b.im * c.im)
    (a * (b * c)).im = a.re * b.re * c.im + a.re * b.im * c.re +
        a.im * b.re * c.re - a.im * b.im * c.im
    int_add_sub_reorder(a.re * b.re * c.im, a.re * b.im * c.re, a.im * b.re * c.re,
        a.im * b.im * c.im)
    a.re * b.re * c.im - a.im * b.im * c.im + a.re * b.im * c.re + a.im * b.re * c.re =
        a.re * b.re * c.im + a.re * b.im * c.re + a.im * b.re * c.re - a.im * b.im * c.im
    ((a * b) * c).im = (a * (b * c)).im
}

/// Multiplication distributes over addition.
theorem gauss_mul_distrib(a: GaussInt, b: GaussInt, c: GaussInt) {
    a * (b + c) = a * b + a * c
} by {
    (a * (b + c)).re = a.re * (b + c).re - a.im * (b + c).im
    (b + c).re = b.re + c.re
    (b + c).im = b.im + c.im
    (a * (b + c)).re = a.re * (b.re + c.re) - a.im * (b.im + c.im)
    a.re * (b.re + c.re) = a.re * b.re + a.re * c.re
    a.im * (b.im + c.im) = a.im * b.im + a.im * c.im
    (a * (b + c)).re = a.re * b.re + a.re * c.re - (a.im * b.im + a.im * c.im)
    int_sub_sum(a.re * b.re + a.re * c.re, a.im * b.im, a.im * c.im)
    a.re * b.re + a.re * c.re - (a.im * b.im + a.im * c.im) =
        a.re * b.re + a.re * c.re - a.im * b.im - a.im * c.im
    (a * (b + c)).re = a.re * b.re + a.re * c.re - a.im * b.im - a.im * c.im
    (a * b + a * c).re = (a * b).re + (a * c).re
    (a * b).re = a.re * b.re - a.im * b.im
    (a * c).re = a.re * c.re - a.im * c.im
    (a * b + a * c).re = a.re * b.re - a.im * b.im + a.re * c.re - a.im * c.im
    int_sub_sub_reorder(a.re * b.re, a.im * b.im, a.re * c.re, a.im * c.im)
    a.re * b.re - a.im * b.im + a.re * c.re - a.im * c.im =
        a.re * b.re + a.re * c.re - a.im * b.im - a.im * c.im
    (a * (b + c)).re = (a * b + a * c).re
    (a * (b + c)).im = a.re * (b + c).im + a.im * (b + c).re
    (a * (b + c)).im = a.re * (b.im + c.im) + a.im * (b.re + c.re)
    a.re * (b.im + c.im) = a.re * b.im + a.re * c.im
    a.im * (b.re + c.re) = a.im * b.re + a.im * c.re
    (a * (b + c)).im = a.re * b.im + a.re * c.im + a.im * b.re + a.im * c.re
    (a * b + a * c).im = (a * b).im + (a * c).im
    (a * b).im = a.re * b.im + a.im * b.re
    (a * c).im = a.re * c.im + a.im * c.re
    (a * b + a * c).im = a.re * b.im + a.im * b.re + a.re * c.im + a.im * c.re
    a.re * b.im + a.im * b.re + a.re * c.im + a.im * c.re =
        a.re * b.im + a.re * c.im + a.im * b.re + a.im * c.re
    (a * (b + c)).im = (a * b + a * c).im
}

/// Multiplication distributes over addition on the right.
theorem gauss_mul_distrib_right(a: GaussInt, b: GaussInt, c: GaussInt) {
    (a + b) * c = a * c + b * c
} by {
    gauss_mul_comm(a + b, c)
    (a + b) * c = c * (a + b)
    gauss_mul_distrib(c, a, b)
    c * (a + b) = c * a + c * b
    gauss_mul_comm(c, a)
    c * a = a * c
    gauss_mul_comm(c, b)
    c * b = b * c
    (a + b) * c = a * c + b * c
}

/// One is the multiplicative identity on the left.
theorem gauss_mul_one_left(a: GaussInt) {
    GaussInt.1 * a = a
} by {
    (GaussInt.1 * a).re = GaussInt.1.re * a.re - GaussInt.1.im * a.im
    GaussInt.1.re = Int.1
    GaussInt.1.im = Int.0
    (GaussInt.1 * a).re = Int.1 * a.re - Int.0 * a.im
    Int.1 * a.re = a.re
    Int.0 * a.im = Int.0
    (GaussInt.1 * a).re = a.re - Int.0
    a.re - Int.0 = a.re
    (GaussInt.1 * a).re = a.re
    (GaussInt.1 * a).im = GaussInt.1.re * a.im + GaussInt.1.im * a.re
    (GaussInt.1 * a).im = Int.1 * a.im + Int.0 * a.re
    Int.1 * a.im = a.im
    Int.0 * a.re = Int.0
    (GaussInt.1 * a).im = a.im + Int.0
    a.im + Int.0 = a.im
    (GaussInt.1 * a).im = a.im
}

/// One is the multiplicative identity on the right.
theorem gauss_mul_one_right(a: GaussInt) {
    a * GaussInt.1 = a
} by {
    (a * GaussInt.1).re = a.re * GaussInt.1.re - a.im * GaussInt.1.im
    GaussInt.1.re = Int.1
    GaussInt.1.im = Int.0
    (a * GaussInt.1).re = a.re * Int.1 - a.im * Int.0
    a.re * Int.1 = a.re
    a.im * Int.0 = Int.0
    (a * GaussInt.1).re = a.re - Int.0
    a.re - Int.0 = a.re
    (a * GaussInt.1).re = a.re
    (a * GaussInt.1).im = a.re * GaussInt.1.im + a.im * GaussInt.1.re
    (a * GaussInt.1).im = a.re * Int.0 + a.im * Int.1
    a.re * Int.0 = Int.0
    a.im * Int.1 = a.im
    (a * GaussInt.1).im = Int.0 + a.im
    Int.0 + a.im = a.im
    (a * GaussInt.1).im = a.im
}

/// Zero is absorbing for multiplication on the right.
theorem gauss_mul_zero_right(a: GaussInt) {
    a * GaussInt.0 = GaussInt.0
} by {
    (a * GaussInt.0).re = a.re * GaussInt.0.re - a.im * GaussInt.0.im
    GaussInt.0.re = Int.0
    GaussInt.0.im = Int.0
    (a * GaussInt.0).re = a.re * Int.0 - a.im * Int.0
    a.re * Int.0 = Int.0
    a.im * Int.0 = Int.0
    (a * GaussInt.0).re = Int.0 - Int.0
    Int.0 - Int.0 = Int.0
    GaussInt.0.re = Int.0
    (a * GaussInt.0).re = GaussInt.0.re
    (a * GaussInt.0).im = a.re * GaussInt.0.im + a.im * GaussInt.0.re
    (a * GaussInt.0).im = a.re * Int.0 + a.im * Int.0
    a.re * Int.0 = Int.0
    a.im * Int.0 = Int.0
    (a * GaussInt.0).im = Int.0 + Int.0
    Int.0 + Int.0 = Int.0
    GaussInt.0.im = Int.0
    (a * GaussInt.0).im = GaussInt.0.im
}

/// Zero is absorbing for multiplication on the left.
theorem gauss_mul_zero_left(a: GaussInt) {
    GaussInt.0 * a = GaussInt.0
} by {
    gauss_mul_comm(GaussInt.0, a)
    GaussInt.0 * a = a * GaussInt.0
    gauss_mul_zero_right(a)
    a * GaussInt.0 = GaussInt.0
    GaussInt.0 * a = GaussInt.0
}

// ============================================================================
// Section 2: conjugation
// ============================================================================

/// The conjugate of a sum is the sum of the conjugates.
theorem gauss_conj_add(a: GaussInt, b: GaussInt) {
    (a + b).conj = a.conj + b.conj
} by {
    (a + b).conj = GaussInt.new((a + b).re, -(a + b).im)
    (a + b).re = a.re + b.re
    (a + b).im = a.im + b.im
    -(a.im + b.im) = -a.im + -b.im
    (a + b).conj = GaussInt.new(a.re + b.re, -a.im + -b.im)
    a.conj = GaussInt.new(a.re, -a.im)
    b.conj = GaussInt.new(b.re, -b.im)
    (a.conj + b.conj).re = a.conj.re + b.conj.re
    (a.conj + b.conj).im = a.conj.im + b.conj.im
    a.conj.re = a.re
    a.conj.im = -a.im
    b.conj.re = b.re
    b.conj.im = -b.im
    (a.conj + b.conj).re = a.re + b.re
    (a.conj + b.conj).im = -a.im + -b.im
    (a + b).conj = a.conj + b.conj
}

/// The conjugate of a product is the product of the conjugates.
theorem gauss_conj_mul(a: GaussInt, b: GaussInt) {
    (a * b).conj = a.conj * b.conj
} by {
    (a * b).re = a.re * b.re - a.im * b.im
    (a * b).im = a.re * b.im + a.im * b.re
    (a * b).conj = GaussInt.new((a * b).re, -((a * b).im))
    a.conj = GaussInt.new(a.re, -a.im)
    b.conj = GaussInt.new(b.re, -b.im)
    (a.conj * b.conj).re = a.conj.re * b.conj.re - a.conj.im * b.conj.im
    (a.conj * b.conj).im = a.conj.re * b.conj.im + a.conj.im * b.conj.re
    a.conj.re = a.re
    a.conj.im = -a.im
    b.conj.re = b.re
    b.conj.im = -b.im
    (a.conj * b.conj).re = a.re * b.re - (-a.im) * (-b.im)
    (a.conj * b.conj).im = a.re * (-b.im) + (-a.im) * b.re
    mul_neg_left(a.im, -b.im)
    (-a.im) * (-b.im) = -(a.im * (-b.im))
    mul_neg_right(a.im, b.im)
    a.im * (-b.im) = -(a.im * b.im)
    neg_neg(a.im * b.im)
    -(-(a.im * b.im)) = a.im * b.im
    (-a.im) * (-b.im) = a.im * b.im
    mul_neg_right(a.re, b.im)
    a.re * (-b.im) = -(a.re * b.im)
    mul_neg_left(a.im, b.re)
    (-a.im) * b.re = -(a.im * b.re)
    (a.conj * b.conj).re = a.re * b.re - a.im * b.im
    (a.conj * b.conj).im = -(a.re * b.im) + -(a.im * b.re)
    neg_distrib(a.re * b.im, a.im * b.re)
    -(a.re * b.im + a.im * b.re) = -(a.re * b.im) + -(a.im * b.re)
    (a.conj * b.conj).im = -((a * b).im)
    (a * b).conj = a.conj * b.conj
}

/// Applying conjugation twice returns the original Gaussian integer.
theorem gauss_conj_conj(a: GaussInt) {
    a.conj.conj = a
} by {
    a.conj = GaussInt.new(a.re, -a.im)
    a.conj.conj = GaussInt.new(a.conj.re, -(a.conj.im))
    a.conj.re = a.re
    a.conj.im = -a.im
    -(-a.im) = a.im
    a.conj.conj = GaussInt.new(a.re, a.im)
}

// ============================================================================
// Section 3: integer square identities
// ============================================================================

/// The embedding of natural numbers into the integers is injective.
theorem int_from_nat_inj(m: Nat, n: Nat) {
    Int.from_nat(m) = Int.from_nat(n) implies m = n
} by {
    if Int.from_nat(m) = Int.from_nat(n) {
        abs_from_nat(m)
        abs(Int.from_nat(m)) = m
        abs_from_nat(n)
        abs(Int.from_nat(n)) = n
        abs(Int.from_nat(m)) = abs(Int.from_nat(n))
        m = n
    }
}

/// A natural number whose square is zero is zero.
theorem nat_sq_zero(x: Nat) {
    x * x = Nat.0 implies x = Nat.0
} by {
    if x * x = Nat.0 {
        if x != Nat.0 {
            pos_of_ne_zero(x)
            Nat.0 < x
            lt_imp_lte_suc(Nat.0, x)
            Nat.1 <= x
            lte_mul(x, x)
            x <= x * x
            lte_trans(Nat.1, x, x * x)
            Nat.1 <= x * x
            Nat.1 <= Nat.0
            not_lt_zero(Nat.1)
            false
        }
        x = Nat.0
    }
}

/// A sum of two squares of naturals is zero only when both are zero.
theorem nat_sq_sum_zero(x: Nat, y: Nat) {
    x * x + y * y = Nat.0 implies x = Nat.0 and y = Nat.0
} by {
    if x * x + y * y = Nat.0 {
        x * x <= x * x + y * y
        x * x <= Nat.0
        if x * x != Nat.0 {
            pos_of_ne_zero(x * x)
            Nat.0 < x * x
            lt_imp_lte_suc(Nat.0, x * x)
            Nat.1 <= x * x
            lte_trans(Nat.1, x * x, Nat.0)
            Nat.1 <= Nat.0
            not_lt_zero(Nat.1)
            false
        }
        x * x = Nat.0
        nat_sq_zero(x)
        x = Nat.0
        y * y <= x * x + y * y
        y * y <= Nat.0
        if y * y != Nat.0 {
            pos_of_ne_zero(y * y)
            Nat.0 < y * y
            lt_imp_lte_suc(Nat.0, y * y)
            Nat.1 <= y * y
            lte_trans(Nat.1, y * y, Nat.0)
            Nat.1 <= Nat.0
            not_lt_zero(Nat.1)
            false
        }
        y * y = Nat.0
        nat_sq_zero(y)
        y = Nat.0
        x = Nat.0 and y = Nat.0
    }
}

/// A sum of two squares of naturals equals one exactly when one is one and
/// the other is zero.
theorem nat_sq_sum_one_cases(x: Nat, y: Nat) {
    x * x + y * y = Nat.1 implies
        (x = Nat.0 and y = Nat.1) or (x = Nat.1 and y = Nat.0)
} by {
    if x * x + y * y = Nat.1 {
        if x * x = Nat.0 {
            nat_sq_zero(x)
            x = Nat.0
            Nat.0 + y * y = Nat.1
            y * y = Nat.1
            nat_mul_eq_one(y, y)
            y = Nat.1
            (x = Nat.0 and y = Nat.1) or (x = Nat.1 and y = Nat.0)
        } else {
            x * x != Nat.0
            pos_of_ne_zero(x * x)
            Nat.0 < x * x
            lt_imp_lte_suc(Nat.0, x * x)
            Nat.1 <= x * x
            x * x <= x * x + y * y
            lte_trans(Nat.1, x * x, x * x + y * y)
            Nat.1 <= x * x + y * y
            x * x + y * y <= x * x + y * y
            lte_antisymm(Nat.1, x * x + y * y)
            Nat.1 = x * x + y * y
            lte_antisymm(x * x, Nat.1)
            x * x <= Nat.1
            lte_antisymm(Nat.1, x * x)
            x * x = Nat.1
            nat_mul_eq_one(x, x)
            x = Nat.1
            Nat.1 * Nat.1 + y * y = Nat.1
            Nat.1 + y * y = Nat.1
            add_cancels_left(Nat.1, y * y, Nat.0)
            y * y = Nat.0
            nat_sq_zero(y)
            y = Nat.0
            (x = Nat.0 and y = Nat.1) or (x = Nat.1 and y = Nat.0)
        }
    }
}

/// A four-term sum with the last two terms moved to the front.
theorem int_add_four_last_next_to_first(a: Int, b: Int, c: Int, d: Int) {
    a + b + c + d = (a + d) + (b + c)
} by {
    a + b + c + d = ((a + b) + c) + d
    ((a + b) + c) + d = (a + b) + (c + d)
    (a + b) + (c + d) = a + (b + (c + d))
    b + (c + d) = (b + c) + d
    a + ((b + c) + d) = a + (d + (b + c))
    a + (d + (b + c)) = (a + d) + (b + c)
}

/// Reassociating two pairs: (a + b) + (c + d) = (a + c) + (b + d).
theorem int_add_pair_rearrange(a: Int, b: Int, c: Int, d: Int) {
    (a + b) + (c + d) = (a + c) + (b + d)
} by {
    (a + b) + (c + d) = a + b + c + d
    int_add_four_last_next_to_first(a, b, c, d)
    a + b + c + d = (a + d) + (b + c)
    (a + d) + (b + c) = a + (d + (b + c))
    a + (d + (b + c)) = a + (b + (d + c))
    d + c = c + d
    b + (d + c) = b + (c + d)
    a + (b + (d + c)) = a + (b + (c + d))
    b + (c + d) = (b + c) + d
    a + (b + (c + d)) = a + ((b + c) + d)
    (b + c) + d = (c + b) + d
    c + b = b + c
    a + ((b + c) + d) = a + ((c + b) + d)
    (c + b) + d = c + (b + d)
    a + ((c + b) + d) = a + (c + (b + d))
    a + (c + (b + d)) = (a + c) + (b + d)
}

/// The square of a difference expands to four terms.
theorem int_square_sub(x: Int, y: Int) {
    (x - y) * (x - y) = x * x - x * y - y * x + y * y
} by {
    (x - y) * (x - y) = (x - y) * x - (x - y) * y
    (x - y) * x = x * x - y * x
    (x - y) * y = x * y - y * y
    (x - y) * (x - y) = x * x - y * x - (x * y - y * y)
    y * x = x * y
    x * x - y * x - (x * y - y * y) = x * x - x * y - (x * y - y * y)
    x * x - x * y - (x * y - y * y) = x * x - x * y + -(x * y - y * y)
    -(x * y - y * y) = -(x * y) + y * y
    x * x - x * y + -(x * y - y * y) = x * x - x * y + (-(x * y) + y * y)
    x * x - x * y + (-(x * y) + y * y) = x * x - x * y - x * y + y * y
    x * x - x * y - x * y + y * y = x * x - x * y - y * x + y * y
}

/// The square of a two-term sum expands to four terms.
theorem int_square_add(x: Int, y: Int) {
    (x + y) * (x + y) = x * x + y * x + (x * y + y * y)
} by {
    (x + y) * (x + y) = (x + y) * x + (x + y) * y
    (x + y) * x = x * x + y * x
    (x + y) * y = x * y + y * y
    (x + y) * (x + y) = x * x + y * x + (x * y + y * y)
}

/// A difference of three terms is a sum of negations.
theorem int_minus_to_add(x: Int, m: Int, n: Int, y: Int) {
    x - m - n + y = x + -m + -n + y
} by {
}

/// A sum of negations regroups as a difference of sums.
theorem int_add_to_minus(x: Int, m: Int, n: Int, y: Int) {
    x + -m + -n + y = (x - m) + (y - n)
} by {
    x + -m + -n + y = (x + -m) + (-n + y)
    (x + -m) + (-n + y) = (x - m) + (y - n)
}

/// Cancelling a term and its negation: x - m + (m + z) = x + z.
theorem int_cancel_sub_add(x: Int, m: Int, z: Int) {
    x - m + (m + z) = x + z
} by {
    x - m = x + -m
    x + -m + (m + z) = x + (-m + (m + z))
    -m + (m + z) = (-m + m) + z
    -m + m = Int.0
    (-m + m) + z = Int.0 + z
    Int.0 + z = z
    x + (-m + (m + z)) = x + z
}

/// Cancelling a swapped term and its negation: x - m + (z + m) = x + z.
theorem int_cancel_swap_add(x: Int, m: Int, z: Int) {
    x - m + (z + m) = x + z
} by {
    z + m = m + z
    x - m + (z + m) = x - m + (m + z)
    int_cancel_sub_add(x, m, z)
    x - m + (m + z) = x + z
}

/// Opposite middle terms cancel in a sum of two expanded squares:
/// (x - m - n + y) + (z + m + n + w) = x + y + z + w.
theorem int_cancel_middle(x: Int, y: Int, z: Int, w: Int, m: Int, n: Int) {
    (x - m - n + y) + (z + m + n + w) = x + y + z + w
} by {
    int_minus_to_add(x, m, n, y)
    x - m - n + y = x + -m + -n + y
    int_add_to_minus(x, m, n, y)
    x + -m + -n + y = (x - m) + (y - n)
    z + m + n + w = (z + m) + (n + w)
    int_add_pair_rearrange(x - m, y - n, z + m, n + w)
    ((x - m) + (y - n)) + ((z + m) + (n + w)) =
        ((x - m) + (z + m)) + ((y - n) + (n + w))
    int_cancel_swap_add(x, m, z)
    x - m + (z + m) = x + z
    int_cancel_sub_add(y, n, w)
    y - n + (n + w) = y + w
    ((x - m) + (z + m)) + ((y - n) + (n + w)) = (x + z) + (y + w)
}

/// The four pure terms of the two expanded squares reassemble into the
/// expansion of the product.
theorem int_pure_reassemble(a: Int, b: Int, c: Int, d: Int) {
    a * c * a * c + b * d * b * d + a * d * a * d + b * c * b * c =
        a * a * c * c + a * a * d * d + b * b * c * c + b * b * d * d
} by {
    a * c * a * c = a * a * c * c
    b * d * b * d = b * b * d * d
    a * d * a * d = a * a * d * d
    b * c * b * c = b * b * c * c
    a * a * c * c + b * b * d * d + a * a * d * d + b * b * c * c =
        (a * a * c * c + b * b * d * d) + (a * a * d * d + b * b * c * c)
    int_add_pair_rearrange(a * a * c * c, b * b * d * d, a * a * d * d, b * b * c * c)
    (a * a * c * c + b * b * d * d) + (a * a * d * d + b * b * c * c) =
        (a * a * c * c + a * a * d * d) + (b * b * d * d + b * b * c * c)
    b * b * d * d + b * b * c * c = b * b * c * c + b * b * d * d
    (a * a * c * c + a * a * d * d) + (b * b * d * d + b * b * c * c) =
        (a * a * c * c + a * a * d * d) + (b * b * c * c + b * b * d * d)
    (a * a * c * c + a * a * d * d) + (b * b * c * c + b * b * d * d) =
        a * a * c * c + a * a * d * d + b * b * c * c + b * b * d * d
}

/// The left-hand side of the two-square identity expands to four pure terms.
theorem int_lhs_expand(a: Int, b: Int, c: Int, d: Int) {
    (a * a + b * b) * (c * c + d * d) =
        a * a * c * c + a * a * d * d + b * b * c * c + b * b * d * d
} by {
    (a * a + b * b) * (c * c + d * d) = (a * a + b * b) * (c * c) + (a * a + b * b) * (d * d)
    (a * a + b * b) * (c * c) = a * a * (c * c) + b * b * (c * c)
    (a * a + b * b) * (d * d) = a * a * (d * d) + b * b * (d * d)
    (a * a + b * b) * (c * c + d * d) =
        a * a * (c * c) + b * b * (c * c) + (a * a * (d * d) + b * b * (d * d))
    a * a * (c * c) + b * b * (c * c) + (a * a * (d * d) + b * b * (d * d)) =
        a * a * (c * c) + (b * b * (c * c) + a * a * (d * d)) + b * b * (d * d)
    b * b * (c * c) + a * a * (d * d) = a * a * (d * d) + b * b * (c * c)
    a * a * (c * c) + (b * b * (c * c) + a * a * (d * d)) + b * b * (d * d) =
        a * a * (c * c) + (a * a * (d * d) + b * b * (c * c)) + b * b * (d * d)
    a * a * (c * c) + (a * a * (d * d) + b * b * (c * c)) + b * b * (d * d) =
        a * a * (c * c) + a * a * (d * d) + b * b * (c * c) + b * b * (d * d)
    a * a * (c * c) + a * a * (d * d) + b * b * (c * c) + b * b * (d * d) =
        a * a * c * c + a * a * d * d + b * b * c * c + b * b * d * d
}

/// The two-square (Brahmagupta-Fibonacci) identity over the integers:
/// (a² + b²)(c² + d²) = (ac - bd)² + (ad + bc)².
theorem int_two_square_identity(a: Int, b: Int, c: Int, d: Int) {
    (a * a + b * b) * (c * c + d * d) =
        (a * c - b * d) * (a * c - b * d) + (a * d + b * c) * (a * d + b * c)
} by {
    int_square_sub(a * c, b * d)
    (a * c - b * d) * (a * c - b * d) =
        a * c * a * c - a * c * b * d - b * d * a * c + b * d * b * d
    int_square_add(a * d, b * c)
    (a * d + b * c) * (a * d + b * c) =
        a * d * a * d + b * c * a * d + (a * d * b * c + b * c * b * c)
    a * d * b * c = a * c * b * d
    b * c * a * d = b * d * a * c
    int_cancel_middle(a * c * a * c, b * d * b * d, a * d * a * d, b * c * b * c,
        a * c * b * d, b * d * a * c)
    a * c * a * c - a * c * b * d - b * d * a * c + b * d * b * d +
        (a * d * a * d + a * c * b * d + b * d * a * c + b * c * b * c) =
        a * c * a * c + b * d * b * d + a * d * a * d + b * c * b * c
    int_pure_reassemble(a, b, c, d)
    a * c * a * c + b * d * b * d + a * d * a * d + b * c * b * c =
        a * a * c * c + a * a * d * d + b * b * c * c + b * b * d * d
    int_lhs_expand(a, b, c, d)
    (a * a + b * b) * (c * c + d * d) =
        a * a * c * c + a * a * d * d + b * b * c * c + b * b * d * d
}

// ============================================================================
// Section 4: the norm and its multiplicativity
// ============================================================================

/// The norm of the imaginary unit is one.
theorem gauss_norm_i {
    GaussInt.i.norm = Int.1
} by {
    GaussInt.i.re = Int.0
    GaussInt.i.im = Int.1
    Int.0 * Int.0 + Int.1 * Int.1 = Int.1
}

/// The norm of one is one.
theorem gauss_norm_one {
    GaussInt.1.norm = Int.1
} by {
    GaussInt.1.re = Int.1
    GaussInt.1.im = Int.0
    Int.1 * Int.1 + Int.0 * Int.0 = Int.1
}

/// The norm of zero is zero.
theorem gauss_norm_zero {
    GaussInt.0.norm = Int.0
} by {
    GaussInt.0.re = Int.0
    GaussInt.0.im = Int.0
    Int.0 * Int.0 + Int.0 * Int.0 = Int.0
}

/// The natural norm of the imaginary unit is one.
theorem gauss_norm_nat_i {
    GaussInt.i.norm_nat = Nat.1
} by {
    GaussInt.i.norm_nat = abs(GaussInt.i.re) * abs(GaussInt.i.re) +
        abs(GaussInt.i.im) * abs(GaussInt.i.im)
    GaussInt.i.re = Int.0
    GaussInt.i.im = Int.1
    GaussInt.i.norm_nat = abs(Int.0) * abs(Int.0) + abs(Int.1) * abs(Int.1)
    Int.0 = Int.from_nat(Nat.0)
    Int.1 = Int.from_nat(Nat.1)
    abs_from_nat(Nat.0)
    abs(Int.from_nat(Nat.0)) = Nat.0
    abs_from_nat(Nat.1)
    abs(Int.from_nat(Nat.1)) = Nat.1
    abs(Int.0) = Nat.0
    abs(Int.1) = Nat.1
    GaussInt.i.norm_nat = Nat.0 * Nat.0 + Nat.1 * Nat.1
    Nat.0 * Nat.0 = Nat.0
    Nat.1 * Nat.1 = Nat.1
    GaussInt.i.norm_nat = Nat.0 + Nat.1
    GaussInt.i.norm_nat = Nat.1
}

/// The natural norm of one is one.
theorem gauss_norm_nat_one {
    GaussInt.1.norm_nat = Nat.1
} by {
    GaussInt.1.norm_nat = abs(GaussInt.1.re) * abs(GaussInt.1.re) +
        abs(GaussInt.1.im) * abs(GaussInt.1.im)
    GaussInt.1.re = Int.1
    GaussInt.1.im = Int.0
    GaussInt.1.norm_nat = abs(Int.1) * abs(Int.1) + abs(Int.0) * abs(Int.0)
    Int.0 = Int.from_nat(Nat.0)
    Int.1 = Int.from_nat(Nat.1)
    abs_from_nat(Nat.0)
    abs(Int.from_nat(Nat.0)) = Nat.0
    abs_from_nat(Nat.1)
    abs(Int.from_nat(Nat.1)) = Nat.1
    abs(Int.0) = Nat.0
    abs(Int.1) = Nat.1
    GaussInt.1.norm_nat = Nat.1 * Nat.1 + Nat.0 * Nat.0
    Nat.0 * Nat.0 = Nat.0
    Nat.1 * Nat.1 = Nat.1
    GaussInt.1.norm_nat = Nat.1 + Nat.0
    GaussInt.1.norm_nat = Nat.1
}

/// The natural norm of zero is zero.
theorem gauss_norm_nat_zero {
    GaussInt.0.norm_nat = Nat.0
} by {
    GaussInt.0.norm_nat = abs(GaussInt.0.re) * abs(GaussInt.0.re) +
        abs(GaussInt.0.im) * abs(GaussInt.0.im)
    GaussInt.0.re = Int.0
    GaussInt.0.im = Int.0
    GaussInt.0.norm_nat = abs(Int.0) * abs(Int.0) + abs(Int.0) * abs(Int.0)
    Int.0 = Int.from_nat(Nat.0)
    abs_from_nat(Nat.0)
    abs(Int.from_nat(Nat.0)) = Nat.0
    abs(Int.0) = Nat.0
    GaussInt.0.norm_nat = Nat.0 * Nat.0 + Nat.0 * Nat.0
    Nat.0 * Nat.0 = Nat.0
    GaussInt.0.norm_nat = Nat.0 + Nat.0
    GaussInt.0.norm_nat = Nat.0
}

/// The square of an integer equals the square of its absolute value,
/// embedded back into the integers.
theorem int_sq_eq_from_nat_abs(a: Int) {
    a * a = Int.from_nat(abs(a) * abs(a))
} by {
    if a = Int.from_nat(abs(a)) {
        let n: Nat = abs(a)
        a * a = Int.from_nat(n) * Int.from_nat(n)
        Int.from_nat(n) * Int.from_nat(n) = Int.from_nat(n * n)
        abs(a) * abs(a) = n * n
        a * a = Int.from_nat(abs(a) * abs(a))
    } else {
        a = -(Int.from_nat(abs(a)))
        let n: Nat = abs(a)
        a * a = -(Int.from_nat(n)) * -(Int.from_nat(n))
        -(Int.from_nat(n)) * -(Int.from_nat(n)) = Int.from_nat(n * n)
        abs(a) * abs(a) = n * n
        a * a = Int.from_nat(abs(a) * abs(a))
    }
}

/// The sum of two integer squares equals the embedded sum of the squares of
/// the absolute values.
theorem int_sq_sum_eq_from_nat_abs(a: Int, b: Int) {
    a * a + b * b = Int.from_nat(abs(a) * abs(a) + abs(b) * abs(b))
} by {
    int_sq_eq_from_nat_abs(a)
    a * a = Int.from_nat(abs(a) * abs(a))
    int_sq_eq_from_nat_abs(b)
    b * b = Int.from_nat(abs(b) * abs(b))
    Int.from_nat(abs(a) * abs(a)) + Int.from_nat(abs(b) * abs(b)) =
        Int.from_nat(abs(a) * abs(a) + abs(b) * abs(b))
    a * a + b * b = Int.from_nat(abs(a) * abs(a) + abs(b) * abs(b))
}

/// The absolute value of a sum of two squares is the sum of the squares of
/// the absolute values.
theorem int_abs_sq_sum(a: Int, b: Int) {
    abs(a * a + b * b) = abs(a) * abs(a) + abs(b) * abs(b)
} by {
    int_sq_sum_eq_from_nat_abs(a, b)
    a * a + b * b = Int.from_nat(abs(a) * abs(a) + abs(b) * abs(b))
    abs_from_nat(abs(a) * abs(a) + abs(b) * abs(b))
    abs(Int.from_nat(abs(a) * abs(a) + abs(b) * abs(b))) =
        abs(a) * abs(a) + abs(b) * abs(b)
    abs(a * a + b * b) = abs(a) * abs(a) + abs(b) * abs(b)
}

/// The norm of a Gaussian integer is the embedding of its natural norm.
theorem gauss_norm_eq_from_nat_norm_nat(a: GaussInt) {
    a.norm = Int.from_nat(a.norm_nat)
} by {
    int_sq_sum_eq_from_nat_abs(a.re, a.im)
    a.re * a.re + a.im * a.im = Int.from_nat(abs(a.re) * abs(a.re) + abs(a.im) * abs(a.im))
    a.norm_nat = abs(a.re) * abs(a.re) + abs(a.im) * abs(a.im)
    a.norm = Int.from_nat(a.norm_nat)
}

/// The natural norm of a product of Gaussian integers is the product of the
/// natural norms.  This is the two-square (Brahmagupta-Fibonacci) identity
/// in disguise: N((a + bi)(c + di)) = (a² + b²)(c² + d²).
theorem gauss_norm_nat_mul(a: GaussInt, b: GaussInt) {
    (a * b).norm_nat = a.norm_nat * b.norm_nat
} by {
    // Expand both sides into the same four-term expression over absolute
    // values of the components.
    (a * b).re = a.re * b.re - a.im * b.im
    (a * b).im = a.re * b.im + a.im * b.re
    (a * b).norm_nat = abs(a.re * b.re - a.im * b.im) * abs(a.re * b.re - a.im * b.im) +
        abs(a.re * b.im + a.im * b.re) * abs(a.re * b.im + a.im * b.re)
    int_abs_sq_sum(a.re * b.re - a.im * b.im, a.re * b.im + a.im * b.re)
    abs((a.re * b.re - a.im * b.im) * (a.re * b.re - a.im * b.im) +
        (a.re * b.im + a.im * b.re) * (a.re * b.im + a.im * b.re)) =
        abs(a.re * b.re - a.im * b.im) * abs(a.re * b.re - a.im * b.im) +
        abs(a.re * b.im + a.im * b.re) * abs(a.re * b.im + a.im * b.re)
    (a * b).norm_nat = abs((a.re * b.re - a.im * b.im) * (a.re * b.re - a.im * b.im) +
        (a.re * b.im + a.im * b.re) * (a.re * b.im + a.im * b.re))
    // The two-square identity over the integers.
    int_two_square_identity(a.re, a.im, b.re, b.im)
    (a.re * b.re - a.im * b.im) * (a.re * b.re - a.im * b.im) +
        (a.re * b.im + a.im * b.re) * (a.re * b.im + a.im * b.re) =
        (a.re * a.re + a.im * a.im) * (b.re * b.re + b.im * b.im)
    abs((a.re * b.re - a.im * b.im) * (a.re * b.re - a.im * b.im) +
        (a.re * b.im + a.im * b.re) * (a.re * b.im + a.im * b.re)) =
        abs((a.re * a.re + a.im * a.im) * (b.re * b.re + b.im * b.im))
    abs_mul(a.re * a.re + a.im * a.im, b.re * b.re + b.im * b.im)
    abs((a.re * a.re + a.im * a.im) * (b.re * b.re + b.im * b.im)) =
        abs(a.re * a.re + a.im * a.im) * abs(b.re * b.re + b.im * b.im)
    int_abs_sq_sum(a.re, a.im)
    abs(a.re * a.re + a.im * a.im) = abs(a.re) * abs(a.re) + abs(a.im) * abs(a.im)
    int_abs_sq_sum(b.re, b.im)
    abs(b.re * b.re + b.im * b.im) = abs(b.re) * abs(b.re) + abs(b.im) * abs(b.im)
    (a * b).norm_nat = abs(a.re * a.re + a.im * a.im) * abs(b.re * b.re + b.im * b.im)
    (a * b).norm_nat = (abs(a.re) * abs(a.re) + abs(a.im) * abs(a.im)) *
        abs(b.re * b.re + b.im * b.im)
    (a * b).norm_nat = (abs(a.re) * abs(a.re) + abs(a.im) * abs(a.im)) *
        (abs(b.re) * abs(b.re) + abs(b.im) * abs(b.im))
    a.norm_nat * b.norm_nat = (abs(a.re) * abs(a.re) + abs(a.im) * abs(a.im)) *
        (abs(b.re) * abs(b.re) + abs(b.im) * abs(b.im))
    (a * b).norm_nat = a.norm_nat * b.norm_nat
}

/// The integer norm of a product of Gaussian integers is the product of the
/// integer norms.
theorem gauss_norm_mul(a: GaussInt, b: GaussInt) {
    (a * b).norm = a.norm * b.norm
} by {
    gauss_norm_nat_mul(a, b)
    (a * b).norm_nat = a.norm_nat * b.norm_nat
    gauss_norm_eq_from_nat_norm_nat(a * b)
    (a * b).norm = Int.from_nat((a * b).norm_nat)
    gauss_norm_eq_from_nat_norm_nat(a)
    gauss_norm_eq_from_nat_norm_nat(b)
    a.norm = Int.from_nat(a.norm_nat)
    b.norm = Int.from_nat(b.norm_nat)
    Int.from_nat(a.norm_nat) * Int.from_nat(b.norm_nat) =
        Int.from_nat(a.norm_nat * b.norm_nat)
    (a * b).norm = a.norm * b.norm
}

/// The norm is invariant under conjugation.
theorem gauss_norm_conj(a: GaussInt) {
    a.conj.norm = a.norm
} by {
    a.conj = GaussInt.new(a.re, -a.im)
    a.conj.re = a.re
    a.conj.im = -a.im
    a.conj.norm = a.re * a.re + (-a.im) * (-a.im)
    (-a.im) * (-a.im) = a.im * a.im
    a.conj.norm = a.re * a.re + a.im * a.im
    a.norm = a.re * a.re + a.im * a.im
    a.conj.norm = a.norm
}

/// The natural norm is invariant under conjugation.
theorem gauss_norm_nat_conj(a: GaussInt) {
    a.conj.norm_nat = a.norm_nat
} by {
    gauss_norm_conj(a)
    a.conj.norm = a.norm
    gauss_norm_eq_from_nat_norm_nat(a.conj)
    a.conj.norm = Int.from_nat(a.conj.norm_nat)
    gauss_norm_eq_from_nat_norm_nat(a)
    a.norm = Int.from_nat(a.norm_nat)
    Int.from_nat(a.conj.norm_nat) = Int.from_nat(a.norm_nat)
    int_from_nat_inj(a.conj.norm_nat, a.norm_nat)
    a.conj.norm_nat = a.norm_nat
}

/// A Gaussian integer times its conjugate is its norm, embedded as a real
/// Gaussian integer.
theorem gauss_mul_conj(a: GaussInt) {
    a * a.conj = GaussInt.new(a.norm, Int.0)
} by {
    (a * a.conj).re = a.re * a.conj.re - a.im * a.conj.im
    (a * a.conj).im = a.re * a.conj.im + a.im * a.conj.re
    a.conj.re = a.re
    a.conj.im = -a.im
    (a * a.conj).re = a.re * a.re - a.im * (-a.im)
    (a * a.conj).im = a.re * (-a.im) + a.im * a.re
    a.im * (-a.im) = -(a.im * a.im)
    a.re * (-a.im) = -(a.re * a.im)
    a.re * a.re - a.im * (-a.im) = a.re * a.re + a.im * a.im
    (a * a.conj).re = a.re * a.re + a.im * a.im
    (a * a.conj).im = -(a.re * a.im) + a.im * a.re
    a.im * a.re = a.re * a.im
    (a * a.conj).im = -(a.re * a.im) + a.re * a.im
    -(a.re * a.im) + a.re * a.im = Int.0
    (a * a.conj).im = Int.0
    a.norm = a.re * a.re + a.im * a.im
    (a * a.conj).re = a.norm
    GaussInt.new(a.norm, Int.0).im = Int.0
    (a * a.conj).im = GaussInt.new(a.norm, Int.0).im
    a * a.conj = GaussInt.new(a.norm, Int.0)
}

/// A Gaussian integer whose natural norm is zero is zero.
theorem gauss_norm_nat_zero_iff_zero(a: GaussInt) {
    a.norm_nat = Nat.0 implies a = GaussInt.0
} by {
    if a.norm_nat = Nat.0 {
        a.norm_nat = abs(a.re) * abs(a.re) + abs(a.im) * abs(a.im)
        abs(a.re) * abs(a.re) + abs(a.im) * abs(a.im) = Nat.0
        nat_sq_sum_zero(abs(a.re), abs(a.im))
        abs(a.re) = Nat.0
        abs(a.im) = Nat.0
        abs_from_nat(Nat.0)
        abs(Int.from_nat(Nat.0)) = Nat.0
        if a.re = Int.from_nat(abs(a.re)) {
            a.re = Int.from_nat(Nat.0)
            a.re = Int.0
        } else {
            a.re = -(Int.from_nat(abs(a.re)))
            a.re = -(Int.from_nat(Nat.0))
            Int.from_nat(Nat.0) = Int.0
            a.re = Int.0
        }
        if a.im = Int.from_nat(abs(a.im)) {
            a.im = Int.from_nat(Nat.0)
            a.im = Int.0
        } else {
            a.im = -(Int.from_nat(abs(a.im)))
            a.im = -(Int.from_nat(Nat.0))
            Int.from_nat(Nat.0) = Int.0
            a.im = Int.0
        }
        a = GaussInt.0
    }
}

/// A nonzero Gaussian integer has positive natural norm.
theorem gauss_norm_nat_pos(a: GaussInt) {
    a != GaussInt.0 implies Nat.0 < a.norm_nat
} by {
    if a != GaussInt.0 {
        if a.norm_nat = Nat.0 {
            gauss_norm_nat_zero_iff_zero(a)
            a = GaussInt.0
            false
        }
        a.norm_nat != Nat.0
        pos_of_ne_zero(a.norm_nat)
        Nat.0 < a.norm_nat
    }
}

// ============================================================================
// Section 5: divisibility and units
// ============================================================================

/// An integer divisor of two integers divides their sum.
theorem int_divides_add(a: Int, b: Int, c: Int) {
    a.divides(b) and a.divides(c) implies a.divides(b + c)
} by {
    if a.divides(b) and a.divides(c) {
        a.divides(b) = exists(d: Int) { d * a = b }
        let (d: Int) satisfy { d * a = b }
        a.divides(c) = exists(e: Int) { e * a = c }
        let (e: Int) satisfy { e * a = c }
        (d + e) * a = d * a + e * a
        (d + e) * a = b + c
        exists(f: Int) { f * a = b + c }
        a.divides(b + c)
    }
}

/// An integer divisor of one factor divides the product.
theorem int_divides_mul(a: Int, b: Int, c: Int) {
    a.divides(b) implies a.divides(b * c)
} by {
    if a.divides(b) {
        a.divides(b) = exists(d: Int) { d * a = b }
        let (d: Int) satisfy { d * a = b }
        (d * c) * a = d * (c * a)
        c * a = a * c
        d * (c * a) = d * (a * c)
        d * (a * c) = d * a * c
        (d * c) * a = d * a * c
        d * a = b
        (d * c) * a = b * c
        exists(f: Int) { f * a = b * c }
        a.divides(b * c)
    }
}

/// An integer divisor of an integer divides its negation.
theorem int_divides_neg(a: Int, b: Int) {
    a.divides(b) implies a.divides(-b)
} by {
    if a.divides(b) {
        a.divides(b) = exists(d: Int) { d * a = b }
        let (d: Int) satisfy { d * a = b }
        (-d) * a = -(d * a)
        (-d) * a = -b
        exists(f: Int) { f * a = -b }
        a.divides(-b)
    }
}

/// A natural prime embeds to an integer prime.
theorem nat_prime_imp_int_prime(p: Nat) {
    p.is_prime implies is_prime(Int.from_nat(p))
} by {
    if p.is_prime {
        is_prime(Int.from_nat(p)) = abs(Int.from_nat(p)).is_prime
        abs_from_nat(p)
        abs(Int.from_nat(p)) = p
        is_prime(Int.from_nat(p)) = p.is_prime
        is_prime(Int.from_nat(p))
    }
}

/// The Gaussian integer embedding of a natural number, n + 0i.
define gauss_of_nat(n: Nat) -> GaussInt {
    GaussInt.new(Int.from_nat(n), Int.0)
}

/// Multiplying a Gaussian integer by the embedding of a natural number
/// scales both components.
theorem gauss_of_nat_mul(n: Nat, z: GaussInt) {
    gauss_of_nat(n) * z = GaussInt.new(Int.from_nat(n) * z.re, Int.from_nat(n) * z.im)
} by {
    gauss_of_nat(n) = GaussInt.new(Int.from_nat(n), Int.0)
    (gauss_of_nat(n) * z).re = gauss_of_nat(n).re * z.re - gauss_of_nat(n).im * z.im
    (gauss_of_nat(n) * z).im = gauss_of_nat(n).re * z.im + gauss_of_nat(n).im * z.re
    gauss_of_nat(n).re = Int.from_nat(n)
    gauss_of_nat(n).im = Int.0
    (gauss_of_nat(n) * z).re = Int.from_nat(n) * z.re - Int.0 * z.im
    (gauss_of_nat(n) * z).im = Int.from_nat(n) * z.im + Int.0 * z.re
    Int.0 * z.im = Int.0
    Int.0 * z.re = Int.0
    (gauss_of_nat(n) * z).re = Int.from_nat(n) * z.re - Int.0
    (gauss_of_nat(n) * z).im = Int.from_nat(n) * z.im + Int.0
    Int.from_nat(n) * z.re - Int.0 = Int.from_nat(n) * z.re
    Int.from_nat(n) * z.im + Int.0 = Int.from_nat(n) * z.im
    (gauss_of_nat(n) * z).re = Int.from_nat(n) * z.re
    (gauss_of_nat(n) * z).im = Int.from_nat(n) * z.im
    gauss_of_nat(n) * z = GaussInt.new(Int.from_nat(n) * z.re, Int.from_nat(n) * z.im)
}

/// A Gaussian integer divisible by the embedding of a natural number has
/// both components divisible by that natural number.
theorem gauss_of_nat_divides_imp_components(p: Nat, z: GaussInt) {
    gauss_of_nat(p).divides(z) implies
        (Int.from_nat(p).divides(z.re) and Int.from_nat(p).divides(z.im))
} by {
    if gauss_of_nat(p).divides(z) {
        gauss_of_nat(p).divides(z) = exists(k: GaussInt) { gauss_of_nat(p) * k = z }
        let (k: GaussInt) satisfy { gauss_of_nat(p) * k = z }
        gauss_of_nat_mul(p, k)
        gauss_of_nat(p) * k = GaussInt.new(Int.from_nat(p) * k.re, Int.from_nat(p) * k.im)
        z = GaussInt.new(Int.from_nat(p) * k.re, Int.from_nat(p) * k.im)
        z.re = Int.from_nat(p) * k.re
        z.im = Int.from_nat(p) * k.im
        exists(d: Int) { d * Int.from_nat(p) = z.re }
        Int.from_nat(p).divides(z.re)
        exists(d: Int) { d * Int.from_nat(p) = z.im }
        Int.from_nat(p).divides(z.im)
        Int.from_nat(p).divides(z.re) and Int.from_nat(p).divides(z.im)
    }
}

/// A Gaussian integer with both components divisible by a natural number is
/// divisible by the embedding of that natural number.
theorem gauss_of_nat_components_imp_divides(p: Nat, z: GaussInt) {
    (Int.from_nat(p).divides(z.re) and Int.from_nat(p).divides(z.im)) implies
        gauss_of_nat(p).divides(z)
} by {
    if Int.from_nat(p).divides(z.re) and Int.from_nat(p).divides(z.im) {
        Int.from_nat(p).divides(z.re)
        Int.from_nat(p).divides(z.im)
        Int.from_nat(p).divides(z.re) = exists(d: Int) { d * Int.from_nat(p) = z.re }
        let (d: Int) satisfy { d * Int.from_nat(p) = z.re }
        Int.from_nat(p).divides(z.im) = exists(e: Int) { e * Int.from_nat(p) = z.im }
        let (e: Int) satisfy { e * Int.from_nat(p) = z.im }
        gauss_of_nat_mul(p, GaussInt.new(d, e))
        gauss_of_nat(p) * GaussInt.new(d, e) =
            GaussInt.new(Int.from_nat(p) * d, Int.from_nat(p) * e)
        Int.from_nat(p) * d = z.re
        Int.from_nat(p) * e = z.im
        gauss_of_nat(p) * GaussInt.new(d, e) = GaussInt.new(z.re, z.im)
        gauss_of_nat(p).divides(z) = exists(k: GaussInt) { gauss_of_nat(p) * k = z }
        gauss_of_nat(p).divides(z)
    }
}

/// A Gaussian integer is divisible by the embedding of a natural number
/// exactly when both components are divisible by that natural number.
theorem gauss_of_nat_divides_iff_components(p: Nat, z: GaussInt) {
    (gauss_of_nat(p).divides(z) =
        (Int.from_nat(p).divides(z.re) and Int.from_nat(p).divides(z.im)))
} by {
    if Int.from_nat(p).divides(z.re) and Int.from_nat(p).divides(z.im) {
        gauss_of_nat_components_imp_divides(p, z)
        gauss_of_nat(p).divides(z)
        (gauss_of_nat(p).divides(z) =
            (Int.from_nat(p).divides(z.re) and Int.from_nat(p).divides(z.im)))
    } else {
        if gauss_of_nat(p).divides(z) {
            gauss_of_nat_divides_imp_components(p, z)
            Int.from_nat(p).divides(z.re) and Int.from_nat(p).divides(z.im)
            false
        }
        (gauss_of_nat(p).divides(z) =
            (Int.from_nat(p).divides(z.re) and Int.from_nat(p).divides(z.im)))
    }
    (gauss_of_nat(p).divides(z) =
        (Int.from_nat(p).divides(z.re) and Int.from_nat(p).divides(z.im)))
}

/// A Gaussian integer with natural norm one is a unit: its inverse is its
/// conjugate.
theorem gauss_unit_of_norm_nat_one(a: GaussInt) {
    a.norm_nat = Nat.1 implies a.is_unit
} by {
    if a.norm_nat = Nat.1 {
        gauss_mul_conj(a)
        a * a.conj = GaussInt.new(a.norm, Int.0)
        gauss_norm_eq_from_nat_norm_nat(a)
        a.norm = Int.from_nat(a.norm_nat)
        a.norm = Int.from_nat(Nat.1)
        Int.from_nat(Nat.1) = Int.1
        a.norm = Int.1
        a * a.conj = GaussInt.new(Int.1, Int.0)
        GaussInt.1 = GaussInt.new(Int.1, Int.0)
        a * a.conj = GaussInt.1
        exists(k: GaussInt) { a * k = GaussInt.1 }
        a.is_unit
    }
}

/// A unit has natural norm one.
theorem gauss_unit_norm_nat_one(a: GaussInt) {
    a.is_unit implies a.norm_nat = Nat.1
} by {
    if a.is_unit {
        a.is_unit = exists(k: GaussInt) { a * k = GaussInt.1 }
        let (k: GaussInt) satisfy { a * k = GaussInt.1 }
        gauss_norm_nat_mul(a, k)
        (a * k).norm_nat = a.norm_nat * k.norm_nat
        gauss_norm_nat_one
        GaussInt.1.norm_nat = Nat.1
        (a * k).norm_nat = Nat.1
        a.norm_nat * k.norm_nat = Nat.1
        nat_mul_eq_one(a.norm_nat, k.norm_nat)
        a.norm_nat = Nat.1 and k.norm_nat = Nat.1
        a.norm_nat = Nat.1
    }
}

/// The units of the Gaussian integers are exactly the elements of norm one.
theorem gauss_unit_iff_norm_nat_one(a: GaussInt) {
    (a.is_unit = (a.norm_nat = Nat.1))
} by {
    if a.is_unit {
        gauss_unit_norm_nat_one(a)
        a.norm_nat = Nat.1
        (a.is_unit = (a.norm_nat = Nat.1))
    } else {
        if a.norm_nat = Nat.1 {
            gauss_unit_of_norm_nat_one(a)
            a.is_unit
            false
        }
        (a.is_unit = (a.norm_nat = Nat.1))
    }
    (a.is_unit = (a.norm_nat = Nat.1))
}

/// The norm of a Gaussian integer is one (as an integer) exactly when it is
/// a unit.
theorem gauss_unit_iff_norm_one(a: GaussInt) {
    (a.is_unit = (a.norm = Int.1))
} by {
    gauss_unit_iff_norm_nat_one(a)
    a.is_unit = (a.norm_nat = Nat.1)
    gauss_norm_eq_from_nat_norm_nat(a)
    a.norm = Int.from_nat(a.norm_nat)
    Int.from_nat(Nat.1) = Int.1
    if a.norm = Int.1 {
        Int.from_nat(a.norm_nat) = Int.1
        Int.from_nat(a.norm_nat) = Int.from_nat(Nat.1)
        int_from_nat_inj(a.norm_nat, Nat.1)
        a.norm_nat = Nat.1
        a.is_unit
        a.is_unit = (a.norm = Int.1)
    } else {
        a.is_unit = (a.norm = Int.1)
    }
    (a.is_unit = (a.norm = Int.1))
}

/// The negation of a constructed Gaussian integer negates both components.
theorem gauss_neg_new(a: Int, b: Int) {
    -(GaussInt.new(a, b)) = GaussInt.new(-a, -b)
} by {
    -(GaussInt.new(a, b)) = GaussInt.new(-(GaussInt.new(a, b)).re, -(GaussInt.new(a, b)).im)
    (GaussInt.new(a, b)).re = a
    (GaussInt.new(a, b)).im = b
    -(GaussInt.new(a, b)) = GaussInt.new(-a, -b)
}

/// The four unit-like elements of the Gaussian integers.
theorem gauss_norm_nat_one_cases(a: GaussInt) {
    a.norm_nat = Nat.1 implies
        a = GaussInt.1 or a = -GaussInt.1 or a = GaussInt.i or a = -GaussInt.i
} by {
    if a.norm_nat = Nat.1 {
        a.norm_nat = abs(a.re) * abs(a.re) + abs(a.im) * abs(a.im)
        abs(a.re) * abs(a.re) + abs(a.im) * abs(a.im) = Nat.1
        nat_sq_sum_one_cases(abs(a.re), abs(a.im))
        if abs(a.re) = Nat.0 and abs(a.im) = Nat.1 {
            abs(a.re) = Nat.0
            abs(a.im) = Nat.1
            if a.re = Int.from_nat(abs(a.re)) {
                a.re = Int.from_nat(Nat.0)
                Int.from_nat(Nat.0) = Int.0
                a.re = Int.0
            } else {
                a.re = -(Int.from_nat(abs(a.re)))
                a.re = -(Int.from_nat(Nat.0))
                Int.from_nat(Nat.0) = Int.0
                a.re = Int.0
            }
            if a.im = Int.from_nat(abs(a.im)) {
                a.im = Int.from_nat(Nat.1)
                Int.from_nat(Nat.1) = Int.1
                a.im = Int.1
                a = GaussInt.new(Int.0, Int.1)
                GaussInt.i = GaussInt.new(Int.0, Int.1)
                a = GaussInt.i
            } else {
                a.im = -(Int.from_nat(abs(a.im)))
                a.im = -(Int.from_nat(Nat.1))
                Int.from_nat(Nat.1) = Int.1
                a.im = -Int.1
                a = GaussInt.new(Int.0, -Int.1)
                GaussInt.i = GaussInt.new(Int.0, Int.1)
                -GaussInt.i = -(GaussInt.new(Int.0, Int.1))
                gauss_neg_new(Int.0, Int.1)
                -(GaussInt.new(Int.0, Int.1)) = GaussInt.new(-Int.0, -Int.1)
                -Int.0 = Int.0
                GaussInt.new(-Int.0, -Int.1) = GaussInt.new(Int.0, -Int.1)
                -GaussInt.i = GaussInt.new(Int.0, -Int.1)
                a = -GaussInt.i
            }
            a = GaussInt.1 or a = -GaussInt.1 or a = GaussInt.i or a = -GaussInt.i
        } else {
            abs(a.re) = Nat.1
            abs(a.im) = Nat.0
            if a.re = Int.from_nat(abs(a.re)) {
                a.re = Int.from_nat(Nat.1)
                Int.from_nat(Nat.1) = Int.1
                a.re = Int.1
            } else {
                a.re = -(Int.from_nat(abs(a.re)))
                a.re = -(Int.from_nat(Nat.1))
                Int.from_nat(Nat.1) = Int.1
                a.re = -Int.1
            }
            if a.im = Int.from_nat(abs(a.im)) {
                a.im = Int.from_nat(Nat.0)
                Int.from_nat(Nat.0) = Int.0
                a.im = Int.0
            } else {
                a.im = -(Int.from_nat(abs(a.im)))
                a.im = -(Int.from_nat(Nat.0))
                Int.from_nat(Nat.0) = Int.0
                a.im = Int.0
            }
            if a.re = Int.1 {
                a = GaussInt.new(Int.1, Int.0)
                GaussInt.1 = GaussInt.new(Int.1, Int.0)
                a = GaussInt.1
            } else {
                a.re = -Int.1
                a = GaussInt.new(-Int.1, Int.0)
                GaussInt.1 = GaussInt.new(Int.1, Int.0)
                -GaussInt.1 = -(GaussInt.new(Int.1, Int.0))
                gauss_neg_new(Int.1, Int.0)
                -(GaussInt.new(Int.1, Int.0)) = GaussInt.new(-Int.1, -Int.0)
                -Int.0 = Int.0
                GaussInt.new(-Int.1, -Int.0) = GaussInt.new(-Int.1, Int.0)
                -GaussInt.1 = GaussInt.new(-Int.1, Int.0)
                a = -GaussInt.1
            }
            a = GaussInt.1 or a = -GaussInt.1 or a = GaussInt.i or a = -GaussInt.i
        }
    }
}

/// Every unit of the Gaussian integers is one of ±1, ±i.
theorem gauss_unit_cases(a: GaussInt) {
    a.is_unit implies a = GaussInt.1 or a = -GaussInt.1 or a = GaussInt.i or a = -GaussInt.i
} by {
    if a.is_unit {
        gauss_unit_norm_nat_one(a)
        a.norm_nat = Nat.1
        gauss_norm_nat_one_cases(a)
        a = GaussInt.1 or a = -GaussInt.1 or a = GaussInt.i or a = -GaussInt.i
    }
}

/// The product of i and -i is one, so i is a unit with inverse -i.
theorem gauss_i_mul_neg_i {
    GaussInt.i * -GaussInt.i = GaussInt.1
} by {
    GaussInt.i = GaussInt.new(Int.0, Int.1)
    -GaussInt.i = -(GaussInt.new(Int.0, Int.1))
    gauss_neg_new(Int.0, Int.1)
    -(GaussInt.new(Int.0, Int.1)) = GaussInt.new(-Int.0, -Int.1)
    -Int.0 = Int.0
    GaussInt.new(-Int.0, -Int.1) = GaussInt.new(Int.0, -Int.1)
    -GaussInt.i = GaussInt.new(Int.0, -Int.1)
    (GaussInt.i * -GaussInt.i).re = GaussInt.i.re * (-GaussInt.i).re - GaussInt.i.im * (-GaussInt.i).im
    (GaussInt.i * -GaussInt.i).im = GaussInt.i.re * (-GaussInt.i).im + GaussInt.i.im * (-GaussInt.i).re
    GaussInt.i.re = Int.0
    GaussInt.i.im = Int.1
    (-GaussInt.i).re = Int.0
    (-GaussInt.i).im = -Int.1
    (GaussInt.i * -GaussInt.i).re = Int.0 * Int.0 - Int.1 * (-Int.1)
    (GaussInt.i * -GaussInt.i).im = Int.0 * (-Int.1) + Int.1 * Int.0
    Int.1 * (-Int.1) = -(Int.1 * Int.1)
    Int.1 * Int.1 = Int.1
    Int.0 * Int.0 = Int.0
    Int.0 * (-Int.1) = Int.0
    Int.1 * Int.0 = Int.0
    (GaussInt.i * -GaussInt.i).re = Int.0 - (-Int.1)
    Int.0 - (-Int.1) = Int.1
    (GaussInt.i * -GaussInt.i).im = Int.0 + Int.0
    Int.0 + Int.0 = Int.0
    (GaussInt.i * -GaussInt.i).re = Int.1
    (GaussInt.i * -GaussInt.i).im = Int.0
    GaussInt.1 = GaussInt.new(Int.1, Int.0)
    GaussInt.i * -GaussInt.i = GaussInt.1
}

/// The square of negative one is one.
theorem gauss_neg_one_mul_neg_one {
    -GaussInt.1 * -GaussInt.1 = GaussInt.1
} by {
    GaussInt.1 = GaussInt.new(Int.1, Int.0)
    -GaussInt.1 = -(GaussInt.new(Int.1, Int.0))
    gauss_neg_new(Int.1, Int.0)
    -(GaussInt.new(Int.1, Int.0)) = GaussInt.new(-Int.1, -Int.0)
    -Int.0 = Int.0
    GaussInt.new(-Int.1, -Int.0) = GaussInt.new(-Int.1, Int.0)
    -GaussInt.1 = GaussInt.new(-Int.1, Int.0)
    (-GaussInt.1 * -GaussInt.1).re = (-GaussInt.1).re * (-GaussInt.1).re - (-GaussInt.1).im * (-GaussInt.1).im
    (-GaussInt.1 * -GaussInt.1).im = (-GaussInt.1).re * (-GaussInt.1).im + (-GaussInt.1).im * (-GaussInt.1).re
    (-GaussInt.1).re = -Int.1
    (-GaussInt.1).im = Int.0
    (-GaussInt.1 * -GaussInt.1).re = (-Int.1) * (-Int.1) - Int.0 * Int.0
    (-GaussInt.1 * -GaussInt.1).im = (-Int.1) * Int.0 + Int.0 * (-Int.1)
    (-Int.1) * (-Int.1) = Int.1
    Int.0 * Int.0 = Int.0
    (-Int.1) * Int.0 = Int.0
    Int.0 * (-Int.1) = Int.0
    (-GaussInt.1 * -GaussInt.1).re = Int.1 - Int.0
    Int.1 - Int.0 = Int.1
    (-GaussInt.1 * -GaussInt.1).im = Int.0 + Int.0
    Int.0 + Int.0 = Int.0
    (-GaussInt.1 * -GaussInt.1).re = Int.1
    (-GaussInt.1 * -GaussInt.1).im = Int.0
    GaussInt.1 = GaussInt.new(Int.1, Int.0)
    -GaussInt.1 * -GaussInt.1 = GaussInt.1
}

// ============================================================================
// Section 6: primes in the Gaussian integers
// ============================================================================

/// A natural prime that divides the product of two Gaussian integers divides
/// one of the factors — a prime congruent to three modulo four stays prime
/// in the Gaussian integers.  The argument runs through the norm: from
/// p | xy the norm identity gives p | N(x)N(y), so p | N(x) or p | N(y);
/// since p ≡ 3 (mod 4), p | a² + b² forces p | a and p | b
/// (`two_squares_descent`), which is exactly p | x in the Gaussian
/// integers.
theorem gauss_prime_three_mod_four_inert(p: Nat, x: GaussInt, y: GaussInt) {
    p.is_prime and p.congr_mod(Nat.3, Nat.4) and gauss_of_nat(p).divides(x * y)
        implies gauss_of_nat(p).divides(x) or gauss_of_nat(p).divides(y)
} by {
    if p.is_prime and p.congr_mod(Nat.3, Nat.4) and gauss_of_nat(p).divides(x * y) {
        gauss_of_nat_divides_imp_components(p, x * y)
        Int.from_nat(p).divides((x * y).re) and Int.from_nat(p).divides((x * y).im)
        Int.from_nat(p).divides((x * y).re)
        Int.from_nat(p).divides((x * y).im)
        int_divides_mul(Int.from_nat(p), (x * y).re, (x * y).re)
        Int.from_nat(p).divides((x * y).re * (x * y).re)
        int_divides_mul(Int.from_nat(p), (x * y).im, (x * y).im)
        Int.from_nat(p).divides((x * y).im * (x * y).im)
        int_divides_add(Int.from_nat(p), (x * y).re * (x * y).re, (x * y).im * (x * y).im)
        Int.from_nat(p).divides((x * y).re * (x * y).re + (x * y).im * (x * y).im)
        (x * y).norm = (x * y).re * (x * y).re + (x * y).im * (x * y).im
        Int.from_nat(p).divides((x * y).norm)
        gauss_norm_mul(x, y)
        (x * y).norm = x.norm * y.norm
        Int.from_nat(p).divides(x.norm * y.norm)
        nat_prime_imp_int_prime(p)
        is_prime(Int.from_nat(p))
        euclids_lemma_prime(Int.from_nat(p), x.norm, y.norm)
        Int.from_nat(p).divides(x.norm) or Int.from_nat(p).divides(y.norm)
        if Int.from_nat(p).divides(x.norm) {
            gauss_norm_eq_from_nat_norm_nat(x)
            x.norm = Int.from_nat(x.norm_nat)
            Int.from_nat(p).divides(Int.from_nat(x.norm_nat))
            div_imp_div_abs(Int.from_nat(p), Int.from_nat(x.norm_nat))
            abs(Int.from_nat(p)).divides(abs(Int.from_nat(x.norm_nat)))
            abs_from_nat(p)
            abs(Int.from_nat(p)) = p
            abs_from_nat(x.norm_nat)
            abs(Int.from_nat(x.norm_nat)) = x.norm_nat
            p.divides(x.norm_nat)
            x.norm_nat = abs(x.re) * abs(x.re) + abs(x.im) * abs(x.im)
            p.divides(abs(x.re) * abs(x.re) + abs(x.im) * abs(x.im))
            two_squares_descent(p, abs(x.re), abs(x.im))
            p.divides(abs(x.re)) and p.divides(abs(x.im))
            p.divides(abs(x.re))
            p.divides(abs(x.im))
            div_from_nat(p, abs(x.re))
            Int.from_nat(p).divides(Int.from_nat(abs(x.re)))
            div_from_nat(p, abs(x.im))
            Int.from_nat(p).divides(Int.from_nat(abs(x.im)))
            if x.re = Int.from_nat(abs(x.re)) {
                Int.from_nat(p).divides(x.re)
            } else {
                x.re = -(Int.from_nat(abs(x.re)))
                int_divides_neg(Int.from_nat(p), Int.from_nat(abs(x.re)))
                Int.from_nat(p).divides(-(Int.from_nat(abs(x.re))))
                Int.from_nat(p).divides(x.re)
            }
            if x.im = Int.from_nat(abs(x.im)) {
                Int.from_nat(p).divides(x.im)
            } else {
                x.im = -(Int.from_nat(abs(x.im)))
                int_divides_neg(Int.from_nat(p), Int.from_nat(abs(x.im)))
                Int.from_nat(p).divides(-(Int.from_nat(abs(x.im))))
                Int.from_nat(p).divides(x.im)
            }
            Int.from_nat(p).divides(x.re) and Int.from_nat(p).divides(x.im)
            gauss_of_nat_components_imp_divides(p, x)
            gauss_of_nat(p).divides(x)
            gauss_of_nat(p).divides(x) or gauss_of_nat(p).divides(y)
        } else {
            Int.from_nat(p).divides(y.norm)
            gauss_norm_eq_from_nat_norm_nat(y)
            y.norm = Int.from_nat(y.norm_nat)
            Int.from_nat(p).divides(Int.from_nat(y.norm_nat))
            div_imp_div_abs(Int.from_nat(p), Int.from_nat(y.norm_nat))
            abs(Int.from_nat(p)).divides(abs(Int.from_nat(y.norm_nat)))
            abs_from_nat(p)
            abs(Int.from_nat(p)) = p
            abs_from_nat(y.norm_nat)
            abs(Int.from_nat(y.norm_nat)) = y.norm_nat
            p.divides(y.norm_nat)
            y.norm_nat = abs(y.re) * abs(y.re) + abs(y.im) * abs(y.im)
            p.divides(abs(y.re) * abs(y.re) + abs(y.im) * abs(y.im))
            two_squares_descent(p, abs(y.re), abs(y.im))
            p.divides(abs(y.re)) and p.divides(abs(y.im))
            p.divides(abs(y.re))
            p.divides(abs(y.im))
            div_from_nat(p, abs(y.re))
            Int.from_nat(p).divides(Int.from_nat(abs(y.re)))
            div_from_nat(p, abs(y.im))
            Int.from_nat(p).divides(Int.from_nat(abs(y.im)))
            if y.re = Int.from_nat(abs(y.re)) {
                Int.from_nat(p).divides(y.re)
            } else {
                y.re = -(Int.from_nat(abs(y.re)))
                int_divides_neg(Int.from_nat(p), Int.from_nat(abs(y.re)))
                Int.from_nat(p).divides(-(Int.from_nat(abs(y.re))))
                Int.from_nat(p).divides(y.re)
            }
            if y.im = Int.from_nat(abs(y.im)) {
                Int.from_nat(p).divides(y.im)
            } else {
                y.im = -(Int.from_nat(abs(y.im)))
                int_divides_neg(Int.from_nat(p), Int.from_nat(abs(y.im)))
                Int.from_nat(p).divides(-(Int.from_nat(abs(y.im))))
                Int.from_nat(p).divides(y.im)
            }
            Int.from_nat(p).divides(y.re) and Int.from_nat(p).divides(y.im)
            gauss_of_nat_components_imp_divides(p, y)
            gauss_of_nat(p).divides(y)
            gauss_of_nat(p).divides(x) or gauss_of_nat(p).divides(y)
        }
        gauss_of_nat(p).divides(x) or gauss_of_nat(p).divides(y)
    }
}

/// A prime congruent to three modulo four is not a sum of two squares, so it
/// is not the norm of any Gaussian integer.
theorem gauss_prime_three_mod_four_not_norm(p: Nat, z: GaussInt) {
    p.is_prime and p.congr_mod(Nat.3, Nat.4) implies z.norm_nat != p
} by {
    if p.is_prime and p.congr_mod(Nat.3, Nat.4) {
        if z.norm_nat = p {
            z.norm_nat = abs(z.re) * abs(z.re) + abs(z.im) * abs(z.im)
            abs(z.re) * abs(z.re) + abs(z.im) * abs(z.im) = p
            exists(a: Nat, b: Nat) { a * a + b * b = p }
            p.congr_mod(Nat.3, Nat.4)
            p.mod(Nat.4) = Nat.3
            p != Nat.2
            prime_sum_two_squares_converse(p)
            p.mod(Nat.4) = Nat.1
            false
        }
        z.norm_nat != p
    }
}

/// A prime congruent to one modulo four is a sum of two squares, hence is
/// the norm of a Gaussian integer (Fermat's two-squares theorem).
theorem gauss_prime_one_mod_four_splits(p: Nat) {
    p.is_prime and p.mod(Nat.4) = Nat.1 implies exists(z: GaussInt) {
        z.norm_nat = p and z.norm = Int.from_nat(p)
    }
} by {
    if p.is_prime and p.mod(Nat.4) = Nat.1 {
        prime_sum_of_two_squares(p)
        exists(a: Nat, b: Nat) { a * a + b * b = p }
        let (a: Nat, b: Nat) satisfy { a * a + b * b = p }
        let z = GaussInt.new(Int.from_nat(a), Int.from_nat(b))
        z.norm_nat = abs(z.re) * abs(z.re) + abs(z.im) * abs(z.im)
        z.re = Int.from_nat(a)
        z.im = Int.from_nat(b)
        abs_from_nat(a)
        abs(Int.from_nat(a)) = a
        abs_from_nat(b)
        abs(Int.from_nat(b)) = b
        z.norm_nat = a * a + b * b
        z.norm_nat = p
        gauss_norm_eq_from_nat_norm_nat(z)
        z.norm = Int.from_nat(z.norm_nat)
        z.norm = Int.from_nat(p)
        z.norm_nat = p and z.norm = Int.from_nat(p)
        exists(w: GaussInt) { w.norm_nat = p and w.norm = Int.from_nat(p) }
    }
}

/// A natural prime is nonzero.
theorem nat_prime_ne_zero(p: Nat) {
    p.is_prime implies p != Nat.0
} by {
    if p.is_prime {
        p.is_prime = (Nat.1 < p and not p.is_composite)
        Nat.1 < p and not p.is_composite
        Nat.1 < p
        lt_suc(Nat.0)
        Nat.0 < Nat.1
        lt_trans(Nat.0, Nat.1, p)
        Nat.0 < p
        if p = Nat.0 {
            Nat.0 < Nat.0
            lt_not_ref(Nat.0)
            false
        }
        p != Nat.0
    }
}

/// A natural prime that is a product of two natural factors is one of them.
theorem nat_prime_product_factor_one(p: Nat, a: Nat, b: Nat) {
    p.is_prime and p = a * b implies a = Nat.1 or b = Nat.1
} by {
    if p.is_prime and p = a * b {
        prime_divides_mul(p, a, b)
        p.divides(a) or p.divides(b)
        if p.divides(a) {
            a.divides(p)
            divides_symm(a, p)
            a = p
            p = a * b
            a = a * b
            p.is_prime
            nat_prime_ne_zero(p)
            p != Nat.0
            a != Nat.0
            a * Nat.1 = a
            a * Nat.1 = a * b
            mul_cancel_left(a, Nat.1, b)
            Nat.1 = b
            b = Nat.1
            a = Nat.1 or b = Nat.1
        } else {
            p.divides(b)
            b.divides(p)
            divides_symm(b, p)
            b = p
            p = a * b
            b = a * b
            p.is_prime
            nat_prime_ne_zero(p)
            p != Nat.0
            b != Nat.0
            b * Nat.1 = b
            b * a = a * b
            a * b = b
            b * a = b
            mul_cancel_left(b, a, Nat.1)
            a = Nat.1
            a = Nat.1 or b = Nat.1
        }
        a = Nat.1 or b = Nat.1
    }
}

/// A Gaussian integer whose natural norm is a natural prime is irreducible:
/// it factors only into a unit and an associate.
theorem gauss_prime_norm_irreducible(z: GaussInt) {
    z.norm_nat.is_prime implies forall(x: GaussInt, y: GaussInt) {
        z = x * y implies x.is_unit or y.is_unit
    }
} by {
    if z.norm_nat.is_prime {
        forall(x: GaussInt, y: GaussInt) {
            if z = x * y {
                gauss_norm_nat_mul(x, y)
                (x * y).norm_nat = x.norm_nat * y.norm_nat
                z = x * y
                z.norm_nat = (x * y).norm_nat
                z.norm_nat = x.norm_nat * y.norm_nat
                z.norm_nat.is_prime
                prime_divides_mul(z.norm_nat, x.norm_nat, y.norm_nat)
                z.norm_nat.divides(x.norm_nat) or z.norm_nat.divides(y.norm_nat)
                if z.norm_nat.divides(x.norm_nat) {
                    z.norm_nat = x.norm_nat * y.norm_nat
                    x.norm_nat.divides(z.norm_nat)
                    divides_symm(x.norm_nat, z.norm_nat)
                    x.norm_nat = z.norm_nat
                    x.norm_nat = x.norm_nat * y.norm_nat
                    z.norm_nat.is_prime
                    nat_prime_ne_zero(z.norm_nat)
                    z.norm_nat != Nat.0
                    x.norm_nat != Nat.0
                    x.norm_nat * Nat.1 = x.norm_nat
                    x.norm_nat * Nat.1 = x.norm_nat * y.norm_nat
                    mul_cancel_left(x.norm_nat, Nat.1, y.norm_nat)
                    Nat.1 = y.norm_nat
                    y.norm_nat = Nat.1
                    gauss_unit_of_norm_nat_one(y)
                    y.is_unit
                    x.is_unit or y.is_unit
                } else {
                    z.norm_nat.divides(y.norm_nat)
                    z.norm_nat = x.norm_nat * y.norm_nat
                    y.norm_nat.divides(z.norm_nat)
                    divides_symm(y.norm_nat, z.norm_nat)
                    y.norm_nat = z.norm_nat
                    y.norm_nat = x.norm_nat * y.norm_nat
                    z.norm_nat.is_prime
                    nat_prime_ne_zero(z.norm_nat)
                    z.norm_nat != Nat.0
                    y.norm_nat != Nat.0
                    y.norm_nat * Nat.1 = y.norm_nat
                    y.norm_nat * x.norm_nat = x.norm_nat * y.norm_nat
                    x.norm_nat * y.norm_nat = y.norm_nat
                    y.norm_nat * x.norm_nat = y.norm_nat
                    mul_cancel_left(y.norm_nat, x.norm_nat, Nat.1)
                    x.norm_nat = Nat.1
                    gauss_unit_of_norm_nat_one(x)
                    x.is_unit
                    x.is_unit or y.is_unit
                }
                x.is_unit or y.is_unit
            }
        }
    }
}

// ============================================================================
// Section 7: least-remainder division for the integers
// ============================================================================

/// A positive integer is its own absolute value, embedded.
theorem int_pos_eq_from_nat_abs(a: Int) {
    a.is_positive implies a = Int.from_nat(abs(a))
} by {
    if a.is_positive {
        neg_or_pos(a)
        a = Int.from_nat(abs(a)) or a = -(Int.from_nat(abs(a)))
        if a = -(Int.from_nat(abs(a))) {
            a.is_positive = (-a).is_negative
            (-a).is_negative
            (-a).is_negative = ((-a) != Int.from_nat(abs(-a)))
            (-a) != Int.from_nat(abs(-a))
            abs_neg(a)
            abs(-a) = abs(a)
            Int.from_nat(abs(-a)) = Int.from_nat(abs(a))
            (-a) != Int.from_nat(abs(a))
            a = -(Int.from_nat(abs(a)))
            neg_neg(Int.from_nat(abs(a)))
            -(-(Int.from_nat(abs(a)))) = Int.from_nat(abs(a))
            -a = Int.from_nat(abs(a))
            false
        }
        a = Int.from_nat(abs(a))
    }
}

/// A nonnegative integer is its own absolute value, embedded.
theorem int_nonneg_eq_from_nat_abs(a: Int) {
    Int.0 <= a implies a = Int.from_nat(abs(a))
} by {
    if Int.0 <= a {
        if a = Int.0 {
            abs(a) = Nat.0
            abs(Int.0) = Nat.0
            Int.from_nat(Nat.0) = Int.0
            a = Int.from_nat(abs(a))
        } else {
            a != Int.0
            Int.0 != a
            a - Int.0 = a
            a.is_positive
            int_pos_eq_from_nat_abs(a)
            a = Int.from_nat(abs(a))
        }
        a = Int.from_nat(abs(a))
    }
}

/// If a natural is not at most another, the other is below it.
theorem nat_not_lte_imp_gt(a: Nat, b: Nat) {
    not (a <= b) implies b < a
} by {
    if not (a <= b) {
        lt_or_lte(b, a)
        if a <= b {
            false
        }
        b < a
    }
}

/// A strict inequality of embedded naturals is a strict inequality of the
/// naturals themselves.
theorem int_lt_from_nat_converse(s: Nat, m: Nat) {
    Int.from_nat(s) < Int.from_nat(m) implies s < m
} by {
    if Int.from_nat(s) < Int.from_nat(m) {
        Int.from_nat(s) < Int.from_nat(m) =
            (Int.from_nat(s) <= Int.from_nat(m) and Int.from_nat(s) != Int.from_nat(m))
        Int.from_nat(s) <= Int.from_nat(m) and Int.from_nat(s) != Int.from_nat(m)
        Int.from_nat(s) <= Int.from_nat(m)
        Int.from_nat(s) != Int.from_nat(m)
        lte_nonnegative_ints_implies_nats(s, m)
        s <= m
        if s = m {
            Int.from_nat(s) = Int.from_nat(m)
            Int.from_nat(s) != Int.from_nat(m)
            false
        }
        s != m
        lt_or_lte(s, m)
        s < m
    }
}

/// Cancelling a common addend on the left preserves an inequality of
/// natural numbers.
theorem nat_lte_cancel_add(a: Nat, b: Nat, c: Nat) {
    a + b <= a + c implies b <= c
} by {
    if a + b <= a + c {
        if c < b {
            lt_suc_right(c, b)
            if c = b {
                c < b
                false
            }
            lt_imp_lte_suc(c, b)
            c.suc <= b
            lt_suc(c)
            c < c.suc
            lte_trans(c, c.suc, b)
            c < b
            a + c < a + b
            a + c <= a + b
            a + b <= a + c
            lte_antisymm(a + b, a + c)
            a + b = a + c
            add_cancels_left(a, b, c)
            b = c
            c < b
            false
        }
        not c < b
        lt_or_lte(c, b)
        b <= c
    }
}

/// Multiplication distributes over truncated subtraction on the left.
theorem nat_sub_distrib_left(a: Nat, b: Nat, c: Nat) {
    a * (b - c) = a * b - a * c
} by {
    if a = Nat.0 {
        Nat.0 * (b - c) = Nat.0
        Nat.0 * b - Nat.0 * c = Nat.0
        Nat.0 * (b - c) = Nat.0 * b - Nat.0 * c
    } else {
        if b < c {
            a * b < a * c
            a * b - a * c = Nat.0
            b - c = Nat.0
            a * Nat.0 = Nat.0
            a * (b - c) = Nat.0
            a * (b - c) = a * b - a * c
        } else {
            not b < c
            lt_or_lte(c, b)
            c <= b
            add_sub(b, c)
            b - c + c = b
            a * (b - c + c) = a * b
            distrib_left(a, b - c, c)
            a * (b - c + c) = a * (b - c) + a * c
            a * (b - c) + a * c = a * b
            add_imp_sub(a * (b - c), a * c, a * b)
            a * b - a * c = a * (b - c)
            a * (b - c) = a * b - a * c
        }
    }
}

/// Rounding up the quotient: when m < 2s and s < m, the complementary
/// remainder m - s satisfies 2(m - s) <= m.
theorem nat_two_mul_sub_le(m: Nat, s: Nat) {
    s < m and m < Nat.2 * s implies Nat.2 * (m - s) <= m
} by {
    if s < m and m < Nat.2 * s {
        lt_imp_lte_suc(m, Nat.2 * s)
        m.suc <= Nat.2 * s
        lt_suc(m)
        m < m.suc
        lte_trans(m, m.suc, Nat.2 * s)
        m <= Nat.2 * s
        lte_add_left(m, m, Nat.2 * s)
        m + m <= m + Nat.2 * s
        s <= m
        lte_mul_both(Nat.2, s, m)
        Nat.2 * s <= Nat.2 * m
        lte_trans(m, Nat.2 * s, Nat.2 * m)
        m <= Nat.2 * m
        lte_add_left(Nat.2 * m, m, Nat.2 * s)
        Nat.2 * m + m <= Nat.2 * m + Nat.2 * s
        m + Nat.2 * s <= Nat.2 * s + m
        lte_trans(m + m, m + Nat.2 * s, Nat.2 * s + m)
        m + m <= Nat.2 * s + m
        lte_add_right(m, m, Nat.2 * s)
        m + m <= Nat.2 * s + m
        Nat.2 * m <= Nat.2 * s + m
        Nat.2 * m = m + m
        Nat.2 * m <= m + Nat.2 * s
        nat_sub_distrib_left(Nat.2, m, s)
        Nat.2 * (m - s) = Nat.2 * m - Nat.2 * s
        lte_mul_both(Nat.2, s, m)
        Nat.2 * s <= Nat.2 * m
        add_sub(Nat.2 * m, Nat.2 * s)
        Nat.2 * m - Nat.2 * s + Nat.2 * s = Nat.2 * m
        add_sub(m, s)
        m - s + s = m
        lte_mul_both(Nat.2, m - s, m)
        Nat.2 * (m - s) <= Nat.2 * m
        add_cancels_left(Nat.2 * s, Nat.2 * (m - s), Nat.2 * m - Nat.2 * s)
        Nat.2 * (m - s) = Nat.2 * m - Nat.2 * s
        Nat.2 * (m - s) + Nat.2 * s <= m + Nat.2 * s
        nat_lte_cancel_add(Nat.2 * s, Nat.2 * (m - s), m)
        Nat.2 * (m - s) <= m
    }
}

/// If twice n is at most a nonzero m, then n is below m.
theorem nat_twice_le_lt(m: Nat, n: Nat) {
    Nat.2 * n <= m and m != Nat.0 implies n < m
} by {
    if Nat.2 * n <= m and m != Nat.0 {
        n <= n + n
        n + n = Nat.2 * n
        n <= Nat.2 * n
        lte_trans(n, Nat.2 * n, m)
        n <= m
        if n = m {
            n = m
            Nat.2 * n <= n
            n <= Nat.2 * n
            lte_antisymm(n, Nat.2 * n)
            n = Nat.2 * n
            n * Nat.1 = n
            n * Nat.2 = Nat.2 * n
            n * Nat.1 = n * Nat.2
            n != Nat.0
            mul_cancel_left(n, Nat.1, Nat.2)
            Nat.1 = Nat.2
            lt_not_ref(Nat.1)
            Nat.1 < Nat.1
            false
        }
        n != m
        lt_or_lte(n, m)
        n < m
    }
}

/// The difference of two embedded naturals embeds the natural difference.
theorem int_from_nat_sub(a: Nat, b: Nat) {
    b <= a implies Int.from_nat(a) - Int.from_nat(b) = Int.from_nat(a - b)
} by {
    if b <= a {
        add_sub(a, b)
        a - b + b = a
        Int.from_nat(a - b) + Int.from_nat(b) = Int.from_nat(a)
        Int.from_nat(a) - Int.from_nat(b) + Int.from_nat(b) = Int.from_nat(a)
        Int.from_nat(a) + (-(Int.from_nat(b))) + Int.from_nat(b) = Int.from_nat(a)
        -(Int.from_nat(b)) + Int.from_nat(b) = Int.0
        Int.from_nat(a) + Int.0 = Int.from_nat(a)
        Int.from_nat(a) - Int.from_nat(b) + Int.from_nat(b) = Int.from_nat(a)
        add_right_cancel(Int.from_nat(a) - Int.from_nat(b), Int.from_nat(a - b), Int.from_nat(b))
        Int.from_nat(a) - Int.from_nat(b) = Int.from_nat(a - b)
    }
}

/// Every integer admits division by a positive modulus with a remainder of
/// least absolute value: a = q·m + r with 2|r| <= m.
theorem int_least_remainder(a: Int, m: Nat) {
    m != Nat.0 implies exists(q: Int, r: Int) {
        a = q * Int.from_nat(m) + r and Nat.2 * abs(r) <= m
    }
} by {
    if m != Nat.0 {
        pos_of_ne_zero(m)
        Nat.0 < m
        from_nat_pos(m)
        Int.from_nat(m).is_positive
        division_theorem(a, Int.from_nat(m))
        exists(q: Int, r: Int) {
            Int.0 <= r and r < Int.from_nat(m) and a = q * Int.from_nat(m) + r
        }
        let (q: Int, r: Int) satisfy {
            Int.0 <= r and r < Int.from_nat(m) and a = q * Int.from_nat(m) + r
        }
        let s: Nat = abs(r)
        int_nonneg_eq_from_nat_abs(r)
        r = Int.from_nat(abs(r))
        r = Int.from_nat(s)
        r < Int.from_nat(m)
        int_lt_from_nat_converse(s, m)
        s < m
        if Nat.2 * s <= m {
            a = q * Int.from_nat(m) + r
            r = Int.from_nat(s)
            abs(r) = s
            abs(Int.from_nat(s)) = s
            Nat.2 * abs(r) = Nat.2 * s
            Nat.2 * abs(r) <= m
            a = q * Int.from_nat(m) + r and Nat.2 * abs(r) <= m
            exists(u: Int, v: Int) {
                a = u * Int.from_nat(m) + v and Nat.2 * abs(v) <= m
            }
        } else {
            nat_not_lte_imp_gt(Nat.2 * s, m)
            m < Nat.2 * s
            nat_two_mul_sub_le(m, s)
            Nat.2 * (m - s) <= m
            s < m
            int_from_nat_sub(m, s)
            Int.from_nat(m) - Int.from_nat(s) = Int.from_nat(m - s)
            neg_sub(Int.from_nat(s), Int.from_nat(m))
            -(Int.from_nat(s) - Int.from_nat(m)) = Int.from_nat(m) - Int.from_nat(s)
            let q2 = q + Int.1
            let r2 = r - Int.from_nat(m)
            r2 = Int.from_nat(s) - Int.from_nat(m)
            q2 * Int.from_nat(m) = (q + Int.1) * Int.from_nat(m)
            (q + Int.1) * Int.from_nat(m) = q * Int.from_nat(m) + Int.1 * Int.from_nat(m)
            Int.1 * Int.from_nat(m) = Int.from_nat(m)
            q2 * Int.from_nat(m) = q * Int.from_nat(m) + Int.from_nat(m)
            q2 * Int.from_nat(m) + r2 = q * Int.from_nat(m) + Int.from_nat(m) + r2
            r2 = Int.from_nat(s) - Int.from_nat(m)
            Int.from_nat(m) + (Int.from_nat(s) - Int.from_nat(m)) = Int.from_nat(s)
            q * Int.from_nat(m) + Int.from_nat(m) + (Int.from_nat(s) - Int.from_nat(m)) =
                q * Int.from_nat(m) + Int.from_nat(s)
            q2 * Int.from_nat(m) + r2 = q * Int.from_nat(m) + Int.from_nat(s)
            q * Int.from_nat(m) + r = q * Int.from_nat(m) + Int.from_nat(s)
            q2 * Int.from_nat(m) + r2 = q * Int.from_nat(m) + r
            a = q2 * Int.from_nat(m) + r2
            r2 = Int.from_nat(s) - Int.from_nat(m)
            abs(r2) = m - s
            abs(Int.from_nat(s) - Int.from_nat(m)) = m - s
            Nat.2 * abs(r2) = Nat.2 * (m - s)
            Nat.2 * abs(r2) <= m
            a = q2 * Int.from_nat(m) + r2 and Nat.2 * abs(r2) <= m
            exists(u: Int, v: Int) {
                a = u * Int.from_nat(m) + v and Nat.2 * abs(v) <= m
            }
        }
        exists(u: Int, v: Int) {
            a = u * Int.from_nat(m) + v and Nat.2 * abs(v) <= m
        }
    }
}

// ============================================================================
// Section 8: the Euclidean division algorithm
// ============================================================================

/// Negation distributes over multiplication on the left.
theorem gauss_mul_neg_left(a: GaussInt, b: GaussInt) {
    (-a) * b = -(a * b)
} by {
    (-a) = GaussInt.new(-(a.re), -(a.im))
    ((-a) * b).re = (-a).re * b.re - (-a).im * b.im
    ((-a) * b).im = (-a).re * b.im + (-a).im * b.re
    (-a).re = -(a.re)
    (-a).im = -(a.im)
    ((-a) * b).re = (-(a.re)) * b.re - (-(a.im)) * b.im
    ((-a) * b).im = (-(a.re)) * b.im + (-(a.im)) * b.re
    mul_neg_left(a.re, b.re)
    (-(a.re)) * b.re = -(a.re * b.re)
    mul_neg_left(a.im, b.im)
    (-(a.im)) * b.im = -(a.im * b.im)
    mul_neg_left(a.re, b.im)
    (-(a.re)) * b.im = -(a.re * b.im)
    mul_neg_left(a.im, b.re)
    (-(a.im)) * b.re = -(a.im * b.re)
    ((-a) * b).re = -(a.re * b.re) - (-(a.im * b.im))
    ((-a) * b).im = -(a.re * b.im) + -(a.im * b.re)
    -(a.re * b.re) - (-(a.im * b.im)) = -(a.re * b.re) + a.im * b.im
    -(a.re * b.re) + a.im * b.im = a.im * b.im - a.re * b.re
    neg_sub(a.re * b.re, a.im * b.im)
    -(a.re * b.re - a.im * b.im) = a.im * b.im - a.re * b.re
    (a * b).re = a.re * b.re - a.im * b.im
    -(a * b).re = -(a.re * b.re - a.im * b.im)
    ((-a) * b).re = -(a * b).re
    neg_distrib(a.re * b.im, a.im * b.re)
    -(a.re * b.im + a.im * b.re) = -(a.re * b.im) + -(a.im * b.re)
    (a * b).im = a.re * b.im + a.im * b.re
    -(a * b).im = -(a.re * b.im + a.im * b.re)
    ((-a) * b).im = -(a * b).im
    -(a * b) = GaussInt.new(-(a * b).re, -(a * b).im)
    ((-a) * b).re = (-(a * b)).re
    ((-a) * b).im = (-(a * b)).im
    (-a) * b = -(a * b)
}

/// Adding constructed Gaussian integers is componentwise.
theorem gauss_add_new(a: Int, b: Int, x: Int, y: Int) {
    GaussInt.new(a, b) + GaussInt.new(x, y) = GaussInt.new(a + x, b + y)
} by {
    (GaussInt.new(a, b) + GaussInt.new(x, y)).re = a + x
    (GaussInt.new(a, b) + GaussInt.new(x, y)).im = b + y
    GaussInt.new(a + x, b + y).re = a + x
    GaussInt.new(a + x, b + y).im = b + y
}

/// If a = b + c then a + -b = c.
theorem int_eq_add_imp_sub(a: Int, b: Int, c: Int) {
    a = b + c implies a + -b = c
} by {
    if a = b + c {
        a + -b = b + c + -b
        b + c + -b = c + b + -b
        c + b + -b = c + (b + -b)
        b + -b = Int.0
        c + Int.0 = c
        c + (b + -b) = c
        a + -b = c
    }
}

/// The square of the doubled bound: 2x <= m implies 4x² <= m².
theorem nat_four_sq_le(m: Nat, x: Nat) {
    Nat.2 * x <= m implies Nat.4 * (x * x) <= m * m
} by {
    if Nat.2 * x <= m {
        lte_mul_both(Nat.2 * x, Nat.2 * x, m)
        (Nat.2 * x) * (Nat.2 * x) <= (Nat.2 * x) * m
        lte_mul_both(m, Nat.2 * x, m)
        m * (Nat.2 * x) <= m * m
        (Nat.2 * x) * m = m * (Nat.2 * x)
        lte_trans((Nat.2 * x) * (Nat.2 * x), (Nat.2 * x) * m, m * m)
        (Nat.2 * x) * (Nat.2 * x) <= m * m
        (Nat.2 * x) * (Nat.2 * x) = Nat.4 * (x * x)
        Nat.4 * (x * x) <= m * m
    }
}

/// Cancelling a common factor preserves an inequality of naturals.
theorem nat_mul_lte_cancel(a: Nat, b: Nat, c: Nat) {
    a != Nat.0 and a * b <= a * c implies b <= c
} by {
    if a != Nat.0 and a * b <= a * c {
        if c < b {
            lt_mul_both(a, c, b)
            a * c < a * b
            a * b <= a * c
            a * c < a * c
            lt_not_ref(a * c)
            false
        }
        not c < b
        lt_or_lte(c, b)
        b <= c
    }
}

/// A Gaussian integer times its conjugate is its norm, embedded.
theorem gauss_mul_conj_of_nat(a: GaussInt) {
    a * a.conj = gauss_of_nat(a.norm_nat)
} by {
    gauss_mul_conj(a)
    a * a.conj = GaussInt.new(a.norm, Int.0)
    gauss_norm_eq_from_nat_norm_nat(a)
    a.norm = Int.from_nat(a.norm_nat)
    GaussInt.new(a.norm, Int.0) = GaussInt.new(Int.from_nat(a.norm_nat), Int.0)
    gauss_of_nat(a.norm_nat) = GaussInt.new(Int.from_nat(a.norm_nat), Int.0)
    a * a.conj = gauss_of_nat(a.norm_nat)
}

/// The Euclidean division algorithm for the Gaussian integers: for any
/// z and nonzero w there are q and r with z = q·w + r and N(r) < N(w).
/// The quotient rounds the components of z·conj(w)/N(w) to the nearest
/// Gaussian integer, which makes the remainder small in norm.
theorem gauss_division(z: GaussInt, w: GaussInt) {
    w != GaussInt.0 implies exists(q: GaussInt, r: GaussInt) {
        z = q * w + r and r.norm_nat < w.norm_nat
    }
} by {
    if w != GaussInt.0 {
        gauss_norm_nat_pos(w)
        Nat.0 < w.norm_nat
        w.norm_nat != Nat.0
        int_least_remainder((z * w.conj).re, w.norm_nat)
        exists(q: Int, r: Int) {
            (z * w.conj).re = q * Int.from_nat(w.norm_nat) + r and
                Nat.2 * abs(r) <= w.norm_nat
        }
        let (q1: Int, r1: Int) satisfy {
            (z * w.conj).re = q1 * Int.from_nat(w.norm_nat) + r1 and
                Nat.2 * abs(r1) <= w.norm_nat
        }
        int_least_remainder((z * w.conj).im, w.norm_nat)
        let (q2: Int, r2: Int) satisfy {
            (z * w.conj).im = q2 * Int.from_nat(w.norm_nat) + r2 and
                Nat.2 * abs(r2) <= w.norm_nat
        }
        // The remainder r = z - q·w has norm_nat · N(w) = |r1|² + |r2|².
        let q = GaussInt.new(q1, q2)
        let r = z + -(q * w)
        gauss_mul_distrib_right(z, -(q * w), w.conj)
        (z + -(q * w)) * w.conj = z * w.conj + (-(q * w)) * w.conj
        gauss_mul_neg_left(q * w, w.conj)
        (-(q * w)) * w.conj = -((q * w) * w.conj)
        (z + -(q * w)) * w.conj = z * w.conj + -((q * w) * w.conj)
        gauss_mul_assoc(q, w, w.conj)
        (q * w) * w.conj = q * (w * w.conj)
        (z + -(q * w)) * w.conj = z * w.conj + -(q * (w * w.conj))
        gauss_mul_conj_of_nat(w)
        w * w.conj = gauss_of_nat(w.norm_nat)
        (z + -(q * w)) * w.conj = z * w.conj + -(q * gauss_of_nat(w.norm_nat))
        gauss_of_nat_mul(w.norm_nat, q)
        q * gauss_of_nat(w.norm_nat) =
            GaussInt.new(Int.from_nat(w.norm_nat) * q.re, Int.from_nat(w.norm_nat) * q.im)
        q.re = q1
        q.im = q2
        q * gauss_of_nat(w.norm_nat) =
            GaussInt.new(Int.from_nat(w.norm_nat) * q1, Int.from_nat(w.norm_nat) * q2)
        gauss_neg_new(Int.from_nat(w.norm_nat) * q1, Int.from_nat(w.norm_nat) * q2)
        -(GaussInt.new(Int.from_nat(w.norm_nat) * q1, Int.from_nat(w.norm_nat) * q2)) =
            GaussInt.new(-(Int.from_nat(w.norm_nat) * q1), -(Int.from_nat(w.norm_nat) * q2))
        gauss_add_new((z * w.conj).re, (z * w.conj).im,
            -(Int.from_nat(w.norm_nat) * q1), -(Int.from_nat(w.norm_nat) * q2))
        GaussInt.new((z * w.conj).re, (z * w.conj).im) +
            GaussInt.new(-(Int.from_nat(w.norm_nat) * q1), -(Int.from_nat(w.norm_nat) * q2)) =
            GaussInt.new((z * w.conj).re + -(Int.from_nat(w.norm_nat) * q1),
                (z * w.conj).im + -(Int.from_nat(w.norm_nat) * q2))
        mul_comm(q1, Int.from_nat(w.norm_nat))
        q1 * Int.from_nat(w.norm_nat) = Int.from_nat(w.norm_nat) * q1
        (z * w.conj).re = q1 * Int.from_nat(w.norm_nat) + r1
        (z * w.conj).re = Int.from_nat(w.norm_nat) * q1 + r1
        int_eq_add_imp_sub((z * w.conj).re, Int.from_nat(w.norm_nat) * q1, r1)
        (z * w.conj).re + -(Int.from_nat(w.norm_nat) * q1) = r1
        mul_comm(q2, Int.from_nat(w.norm_nat))
        q2 * Int.from_nat(w.norm_nat) = Int.from_nat(w.norm_nat) * q2
        (z * w.conj).im = q2 * Int.from_nat(w.norm_nat) + r2
        (z * w.conj).im = Int.from_nat(w.norm_nat) * q2 + r2
        int_eq_add_imp_sub((z * w.conj).im, Int.from_nat(w.norm_nat) * q2, r2)
        (z * w.conj).im + -(Int.from_nat(w.norm_nat) * q2) = r2
        r * w.conj = GaussInt.new(r1, r2)
        gauss_norm_nat_mul(r, w.conj)
        (r * w.conj).norm_nat = r.norm_nat * w.conj.norm_nat
        gauss_norm_nat_conj(w)
        w.conj.norm_nat = w.norm_nat
        (r * w.conj).norm_nat = r.norm_nat * w.norm_nat
        GaussInt.new(r1, r2).norm_nat = abs(r1) * abs(r1) + abs(r2) * abs(r2)
        r.norm_nat * w.norm_nat = abs(r1) * abs(r1) + abs(r2) * abs(r2)
        Nat.2 * abs(r1) <= w.norm_nat
        Nat.2 * abs(r2) <= w.norm_nat
        nat_four_sq_le(w.norm_nat, abs(r1))
        Nat.4 * (abs(r1) * abs(r1)) <= w.norm_nat * w.norm_nat
        nat_four_sq_le(w.norm_nat, abs(r2))
        Nat.4 * (abs(r2) * abs(r2)) <= w.norm_nat * w.norm_nat
        lte_add_left(Nat.4 * (abs(r1) * abs(r1)), Nat.4 * (abs(r2) * abs(r2)), w.norm_nat * w.norm_nat)
        Nat.4 * (abs(r1) * abs(r1)) + Nat.4 * (abs(r2) * abs(r2)) <= w.norm_nat * w.norm_nat + w.norm_nat * w.norm_nat
        w.norm_nat * w.norm_nat + w.norm_nat * w.norm_nat = Nat.2 * (w.norm_nat * w.norm_nat)
        Nat.4 * (abs(r1) * abs(r1)) + Nat.4 * (abs(r2) * abs(r2)) <= Nat.2 * (w.norm_nat * w.norm_nat)
        distrib_left(Nat.4, abs(r1) * abs(r1), abs(r2) * abs(r2))
        Nat.4 * ((abs(r1) * abs(r1)) + (abs(r2) * abs(r2))) =
            Nat.4 * (abs(r1) * abs(r1)) + Nat.4 * (abs(r2) * abs(r2))
        Nat.4 * ((abs(r1) * abs(r1)) + (abs(r2) * abs(r2))) <= Nat.2 * (w.norm_nat * w.norm_nat)
        Nat.4 * (abs(r1) * abs(r1) + abs(r2) * abs(r2)) <= Nat.2 * (w.norm_nat * w.norm_nat)
        abs(r1) * abs(r1) + abs(r2) * abs(r2) = r.norm_nat * w.norm_nat
        Nat.4 * (r.norm_nat * w.norm_nat) <= Nat.2 * (w.norm_nat * w.norm_nat)
        Nat.4 * (r.norm_nat * w.norm_nat) = Nat.2 * (Nat.2 * (r.norm_nat * w.norm_nat))
        Nat.2 * (Nat.2 * (r.norm_nat * w.norm_nat)) <= Nat.2 * (w.norm_nat * w.norm_nat)
        Nat.2 != Nat.0
        mul_cancel_left(Nat.2, Nat.2 * (r.norm_nat * w.norm_nat), w.norm_nat * w.norm_nat)
        Nat.2 * (r.norm_nat * w.norm_nat) <= w.norm_nat * w.norm_nat
        r.norm_nat * w.norm_nat = w.norm_nat * r.norm_nat
        Nat.2 * (w.norm_nat * r.norm_nat) <= w.norm_nat * w.norm_nat
        nat_mul_lte_cancel(w.norm_nat, Nat.2 * r.norm_nat, w.norm_nat)
        Nat.2 * r.norm_nat <= w.norm_nat
        nat_twice_le_lt(w.norm_nat, r.norm_nat)
        r.norm_nat < w.norm_nat
        // z = q·w + r
        gauss_add_assoc(q * w, z, -(q * w))
        q * w + (z + -(q * w)) = q * w + z + -(q * w)
        gauss_add_comm(q * w, z)
        q * w + z = z + q * w
        q * w + (z + -(q * w)) = z + q * w + -(q * w)
        gauss_add_assoc(z, q * w, -(q * w))
        z + q * w + -(q * w) = z + (q * w + -(q * w))
        gauss_add_neg(q * w)
        q * w + -(q * w) = GaussInt.0
        z + (q * w + -(q * w)) = z + GaussInt.0
        gauss_add_zero_right(z)
        z + GaussInt.0 = z
        q * w + (z + -(q * w)) = z
        r = z + -(q * w)
        q * w + r = z
        z = q * w + r
        z = q * w + r and r.norm_nat < w.norm_nat
        exists(u: GaussInt, v: GaussInt) {
            z = u * w + v and v.norm_nat < w.norm_nat
        }
    }
}
