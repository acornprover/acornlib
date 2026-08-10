/// Lattice-point counting.
///
/// The number of integer lattice points in geometric regions: exact counts for
/// rectangles and triangles, and the connection between lattice points on a
/// circle and representations as a sum of two squares.
from nat import Nat, lt_or_lte, not_lt_zero, lt_diff, lt_suc_right, lt_suc, lte_ref,
    lte_and_lt, lte_trans, lte_mul_both, lte_mul_right, lt_mul_both, lte_add_left, lte_add_right,
    add_comm, add_cancels_left, add_cancels_right, add_sub, mul_comm, mul_one_right,
    pos_of_ne_zero, lt_imp_lte_suc, lte_antisymm, suc_ne, add_to_zero, sum_lte,
    mul_two_left, div_mod_decomp, div_mul, add_identity_right, sub_lt, add_imp_sub,
    lt_add_suc, lt_not_ref
from pair import Pair, pair_eta, pair_new_first, pair_new_second
from list import List, map, map_length, length_range, map_contains, map_contains_of_contains,
    range_is_unique, range_contains_of_lt, lt_of_range_contains, injective_map_is_unique
from data.basic.functions import is_injective_fn
from finite_set import FiniteSet, fs_from_list, finite_set_ext, finite_set_ext_contains,
    fs_insert, fs_union, finite_set_singleton_contains_eq,
    finite_set_from_unique_list_cardinality_is_length,
    finite_set_disjoint_union_cardinality_is, finite_set_union_contains_eq
from data.finite.finite_set_card import fs_card, fs_card_eq_of_cardinality_is, fs_card_cardinality_is,
    fs_card_singleton
from data.finite.finite_set_filter import finite_set_filter, finite_set_filter_contains_eq,
    finite_set_filter_subset
from data.finite.finite_set_product import finite_set_product, finite_set_product_contains_eq
from data.finite.finite_set_product_card import fs_card_product
from data.finite.finite_set_card_bounds import fs_card_le_of_subset_le
from data.finite.finite_set_card_compare import fs_card_mono
from data.finite.finite_set_subset_intro import fs_subset_eq_intro
from data.finite.finite_set_membership import fs_from_list_contains_eq
from data.nat.nat_range_set import range_set, range_set_contains_eq, range_set_card
from combinatorics import triangular, triangular_zero, triangular_suc, triangular_doubled
from number_theory.sum_of_two_squares import prime_sum_of_two_squares
from int import Int, sub_nat, sub_nat_cancel_right, abs, abs_from_nat, abs_neg,
    neg_or_pos, neg_zero, sub_nat_self, sub_nat_add_left

numerals Nat

// ============================================================================
// Auxiliary natural-number lemmas.
// ============================================================================

/// Zero is at most every natural.
theorem zero_lte(a: Nat) {
    Nat.0 <= a
} by {
    lt_or_lte(a, Nat.0)
    a < Nat.0 or Nat.0 <= a
    not_lt_zero(a)
    not (a < Nat.0)
    Nat.0 <= a
}

/// Strictly smaller naturals are at most.
theorem lt_imp_lte(a: Nat, b: Nat) {
    a < b implies a <= b
} by {
    if a < b {
        lt_diff(a, b)
        exists(c: Nat) { a + c = b and c != Nat.0 }
        let c: Nat satisfy { a + c = b and c != Nat.0 }
        a <= b
    }
}

/// A natural is below a successor exactly when it is at most the predecessor.
theorem lt_suc_iff_lte(a: Nat, b: Nat) {
    (a < b.suc) = (a <= b)
} by {
    if a < b.suc {
        lt_suc_right(a, b)
        a = b or a < b
        if a = b {
            lte_ref(a)
            a <= b
        }
        if a < b {
            lt_imp_lte(a, b)
            a <= b
        }
        a <= b
    }
    (a < b.suc) implies (a <= b)
    if a <= b {
        lt_suc(b)
        b < b.suc
        lte_and_lt(a, b, b.suc)
        a < b.suc
    }
    (a <= b) implies (a < b.suc)
    (a < b.suc) = (a <= b)
}

/// Squaring is monotone on naturals.
theorem square_mono(a: Nat, b: Nat) {
    a <= b implies a * a <= b * b
} by {
    if a <= b {
        lte_mul_both(a, a, b)
        a * a <= a * b
        lte_mul_right(b, a, b)
        a * b <= b * b
        lte_trans(a * a, a * b, b * b)
        a * a <= b * b
    }
}

/// Squaring reflects order on naturals.
theorem square_le_cancel(a: Nat, b: Nat) {
    a * a <= b * b implies a <= b
} by {
    if a * a <= b * b {
        if b < a {
            a != Nat.0
            lt_mul_both(a, b, a)
            a * b < a * a
            b <= a
            lte_mul_both(b, b, a)
            b * b <= b * a
            b * a = a * b
            b * b <= a * b
            lte_and_lt(b * b, a * b, a * a)
            b * b < a * a
            lte_and_lt(a * a, b * b, a * a)
            a * a < a * a
            false
        }
        a <= b
    }
}

/// A sum bounded by a natural bounds its left summand.
theorem add_lte_imp_left_lte(a: Nat, b: Nat, n: Nat) {
    a + b <= n implies a <= n
} by {
    if a + b <= n {
        zero_lte(b)
        lte_add_left(a, Nat.0, b)
        a + Nat.0 <= a + b
        a + Nat.0 = a
        a <= a + b
        lte_trans(a, a + b, n)
        a <= n
    }
}

/// A sum bounded by a natural bounds its right summand.
theorem add_lte_imp_right_lte(a: Nat, b: Nat, n: Nat) {
    a + b <= n implies b <= n
} by {
    if a + b <= n {
        add_comm(a, b)
        a + b = b + a
        add_lte_imp_left_lte(b, a, n)
        b <= n
    }
}

/// If a sum of squares is at most a square, the first summand's base is at most the root.
theorem sum_sq_le_imp_first_lte(x: Nat, y: Nat, n: Nat) {
    x * x + y * y <= n * n implies x <= n
} by {
    if x * x + y * y <= n * n {
        zero_lte(y * y)
        lte_add_left(x * x, Nat.0, y * y)
        x * x + Nat.0 <= x * x + y * y
        x * x + Nat.0 = x * x
        x * x <= x * x + y * y
        lte_trans(x * x, x * x + y * y, n * n)
        x * x <= n * n
        square_le_cancel(x, n)
        x <= n
    }
}

/// If a sum of squares is at most a square, the second summand's base is at most the root.
theorem sum_sq_le_imp_second_lte(x: Nat, y: Nat, n: Nat) {
    x * x + y * y <= n * n implies y <= n
} by {
    if x * x + y * y <= n * n {
        add_comm(x * x, y * y)
        x * x + y * y = y * y + x * x
        sum_sq_le_imp_first_lte(y, x, n)
        y <= n
    }
}

/// If a sum of squares equals a square, the first summand's base is at most the root.
theorem sum_sq_eq_imp_first_lte(x: Nat, y: Nat, n: Nat) {
    x * x + y * y = n * n implies x <= n
} by {
    if x * x + y * y = n * n {
        zero_lte(y * y)
        lte_add_left(x * x, Nat.0, y * y)
        x * x + Nat.0 <= x * x + y * y
        x * x + Nat.0 = x * x
        x * x <= x * x + y * y
        x * x + y * y = n * n
        x * x + y * y <= n * n
        lte_trans(x * x, x * x + y * y, n * n)
        x * x <= n * n
        square_le_cancel(x, n)
        x <= n
    }
}

/// If a sum of squares equals a square, the second summand's base is at most the root.
theorem sum_sq_eq_imp_second_lte(x: Nat, y: Nat, n: Nat) {
    x * x + y * y = n * n implies y <= n
} by {
    if x * x + y * y = n * n {
        add_comm(x * x, y * y)
        x * x + y * y = y * y + x * x
        sum_sq_eq_imp_first_lte(y, x, n)
        y <= n
    }
}

// ============================================================================
// The exact rectangle count.
// ============================================================================

/// The rectangle of lattice points with x ≤ m and y ≤ n.
define rectangle_points(m: Nat, n: Nat) -> FiniteSet[Pair[Nat, Nat]] {
    finite_set_product(range_set(m.suc), range_set(n.suc))
}

/// A pair (x, y) lies in the rectangle exactly when x ≤ m and y ≤ n.
theorem rectangle_points_contains_eq(m: Nat, n: Nat, x: Nat, y: Nat) {
    rectangle_points(m, n).contains(Pair.new(x, y)) = (x <= m and y <= n)
} by {
    rectangle_points(m, n) = finite_set_product(range_set(m.suc), range_set(n.suc))
    pair_new_first(x, y)
    Pair.new(x, y).first = x
    pair_new_second(x, y)
    Pair.new(x, y).second = y
    finite_set_product_contains_eq(range_set(m.suc), range_set(n.suc), Pair.new(x, y))
    finite_set_product(range_set(m.suc), range_set(n.suc)).contains(Pair.new(x, y)) =
        (range_set(m.suc).contains(x) and range_set(n.suc).contains(y))
    rectangle_points(m, n).contains(Pair.new(x, y)) =
        (range_set(m.suc).contains(x) and range_set(n.suc).contains(y))
    range_set_contains_eq(m.suc, x)
    range_set(m.suc).contains(x) = (x < m.suc)
    range_set_contains_eq(n.suc, y)
    range_set(n.suc).contains(y) = (y < n.suc)
    lt_suc_iff_lte(x, m)
    (x < m.suc) = (x <= m)
    lt_suc_iff_lte(y, n)
    (y < n.suc) = (y <= n)
    rectangle_points(m, n).contains(Pair.new(x, y)) = (x <= m and y <= n)
}

/// The number of lattice points (x, y) with x ≤ m and y ≤ n is (m+1)(n+1).
theorem rectangle_points_card(m: Nat, n: Nat) {
    fs_card(rectangle_points(m, n)) = (m + Nat.1) * (n + Nat.1)
} by {
    rectangle_points(m, n) = finite_set_product(range_set(m.suc), range_set(n.suc))
    range_set_card(m.suc)
    fs_card(range_set(m.suc)) = m.suc
    range_set_card(n.suc)
    fs_card(range_set(n.suc)) = n.suc
    fs_card_product(range_set(m.suc), range_set(n.suc))
    fs_card(finite_set_product(range_set(m.suc), range_set(n.suc))) =
        fs_card(range_set(n.suc)) * fs_card(range_set(m.suc))
    fs_card(rectangle_points(m, n)) = n.suc * m.suc
    fs_card(rectangle_points(m, n)) = (m + Nat.1) * (n + Nat.1)
}

// ============================================================================
// More auxiliary natural-number lemmas.
// ============================================================================

/// A natural at most zero is zero.
theorem lte_zero_imp_zero(a: Nat) {
    a <= Nat.0 implies a = Nat.0
} by {
    if a <= Nat.0 {
        zero_lte(a)
        Nat.0 <= a
        lte_antisymm(a, Nat.0)
        a = Nat.0
    }
}

/// At most but distinct means strictly smaller.
theorem lte_neq_imp_lt(a: Nat, b: Nat) {
    a <= b and a != b implies a < b
} by {
    if a <= b and a != b {
        let d: Nat satisfy { a + d = b }
        if d = Nat.0 {
            a = b
            false
        }
        d != Nat.0
        a < b
    }
}

/// A small square is the square of a small number.
theorem sq_le_imp_le(x: Nat, n: Nat) {
    x * x <= n implies x <= n
} by {
    if x * x <= n {
        if x = Nat.0 {
            zero_lte(n)
            Nat.0 <= n
            x <= n
        }
        if x != Nat.0 {
            pos_of_ne_zero(x)
            Nat.0 < x
            lt_imp_lte_suc(Nat.0, x)
            Nat.1 <= x
            lte_mul_both(x, Nat.1, x)
            x * Nat.1 <= x * x
            x * Nat.1 = x
            x <= x * x
            lte_trans(x, x * x, n)
            x <= n
        }
        x <= n
    }
}

/// An equality of a sum with a natural bounds the left summand.
theorem add_eq_imp_left_lte(a: Nat, b: Nat, n: Nat) {
    a + b = n implies a <= n
} by {
    if a + b = n {
        zero_lte(b)
        lte_add_left(a, Nat.0, b)
        a + Nat.0 <= a + b
        a + Nat.0 = a
        a <= a + b
        a + b = n
        a + b <= n
        lte_trans(a, a + b, n)
        a <= n
    }
}

/// An equality of a sum with a natural bounds the right summand.
theorem add_eq_imp_right_lte(a: Nat, b: Nat, n: Nat) {
    a + b = n implies b <= n
} by {
    if a + b = n {
        add_comm(a, b)
        a + b = b + a
        add_eq_imp_left_lte(b, a, n)
        b <= n
    }
}

/// A natural is at most a successor exactly when it is at most or equal to the successor.
theorem lte_suc_or_eq(a: Nat, b: Nat) {
    (a <= b.suc) = (a <= b or a = b.suc)
} by {
    if a <= b.suc {
        if a = b.suc {
            a <= b or a = b.suc
        }
        if a != b.suc {
            lte_neq_imp_lt(a, b.suc)
            a < b.suc
            lt_suc_right(a, b)
            a = b or a < b
            if a = b {
                lte_ref(a)
                a <= b
            }
            if a < b {
                lt_imp_lte(a, b)
                a <= b
            }
            a <= b
            a <= b or a = b.suc
        }
        a <= b or a = b.suc
        (a <= b.suc) = (a <= b or a = b.suc)
    }
    if not (a <= b.suc) {
        if a <= b or a = b.suc {
            if a <= b {
                lt_suc(b)
                b < b.suc
                lte_and_lt(a, b, b.suc)
                a < b.suc
                lt_imp_lte(a, b.suc)
                a <= b.suc
                false
            }
            if a = b.suc {
                lte_ref(b.suc)
                a <= b.suc
                false
            }
            false
        }
        not (a <= b or a = b.suc)
        (a <= b.suc) = (a <= b or a = b.suc)
    }
}

// ============================================================================
// The circle and representations as a sum of two squares.
// ============================================================================

/// Predicate: the pair (x, y) lies on the circle of radius r.
define circle_pred(r: Nat, p: Pair[Nat, Nat]) -> Bool {
    p.first * p.first + p.second * p.second = r * r
}

/// The lattice points on the circle x² + y² = r².
define circle_on_points(r: Nat) -> FiniteSet[Pair[Nat, Nat]] {
    finite_set_filter(finite_set_product(range_set(r.suc), range_set(r.suc)), circle_pred(r))
}

/// A pair (x, y) lies on the circle x² + y² = r² exactly when x² + y² = r².
theorem circle_on_points_contains_eq(r: Nat, x: Nat, y: Nat) {
    circle_on_points(r).contains(Pair.new(x, y)) = (x * x + y * y = r * r)
} by {
    circle_on_points(r) =
        finite_set_filter(finite_set_product(range_set(r.suc), range_set(r.suc)), circle_pred(r))
    pair_new_first(x, y)
    Pair.new(x, y).first = x
    pair_new_second(x, y)
    Pair.new(x, y).second = y
    finite_set_filter_contains_eq(finite_set_product(range_set(r.suc), range_set(r.suc)),
        circle_pred(r), Pair.new(x, y))
    finite_set_filter(finite_set_product(range_set(r.suc), range_set(r.suc)), circle_pred(r)).contains(Pair.new(x, y)) =
        (finite_set_product(range_set(r.suc), range_set(r.suc)).contains(Pair.new(x, y))
            and circle_pred(r, Pair.new(x, y)))
    finite_set_product_contains_eq(range_set(r.suc), range_set(r.suc), Pair.new(x, y))
    finite_set_product(range_set(r.suc), range_set(r.suc)).contains(Pair.new(x, y)) =
        (range_set(r.suc).contains(x) and range_set(r.suc).contains(y))
    circle_pred(r, Pair.new(x, y)) =
        (Pair.new(x, y).first * Pair.new(x, y).first +
            Pair.new(x, y).second * Pair.new(x, y).second = r * r)
    circle_pred(r, Pair.new(x, y)) = (x * x + y * y = r * r)
    range_set_contains_eq(r.suc, x)
    range_set(r.suc).contains(x) = (x < r.suc)
    range_set_contains_eq(r.suc, y)
    range_set(r.suc).contains(y) = (y < r.suc)
    lt_suc_iff_lte(x, r)
    (x < r.suc) = (x <= r)
    lt_suc_iff_lte(y, r)
    (y < r.suc) = (y <= r)
    if x * x + y * y = r * r {
        sum_sq_eq_imp_first_lte(x, y, r)
        x <= r
        lt_suc_iff_lte(x, r)
        x < r.suc
        sum_sq_eq_imp_second_lte(x, y, r)
        y <= r
        lt_suc_iff_lte(y, r)
        y < r.suc
    }
    (x * x + y * y = r * r) implies (x < r.suc and y < r.suc)
    ((x < r.suc) and (y < r.suc) and (x * x + y * y = r * r)) =
        (x * x + y * y = r * r)
    circle_on_points(r).contains(Pair.new(x, y)) = (x * x + y * y = r * r)
}

/// Predicate: the pair (x, y) satisfies x² + y² = n.
define square_sum_eq_pred(n: Nat, p: Pair[Nat, Nat]) -> Bool {
    p.first * p.first + p.second * p.second = n
}

/// The representations of n as an ordered sum of two squares of naturals.
define two_square_repr(n: Nat) -> FiniteSet[Pair[Nat, Nat]] {
    finite_set_filter(finite_set_product(range_set(n.suc), range_set(n.suc)), square_sum_eq_pred(n))
}

/// A pair (x, y) is a representation of n as a sum of two squares exactly when x² + y² = n.
theorem two_square_repr_contains_eq(n: Nat, x: Nat, y: Nat) {
    two_square_repr(n).contains(Pair.new(x, y)) = (x * x + y * y = n)
} by {
    two_square_repr(n) =
        finite_set_filter(finite_set_product(range_set(n.suc), range_set(n.suc)), square_sum_eq_pred(n))
    pair_new_first(x, y)
    Pair.new(x, y).first = x
    pair_new_second(x, y)
    Pair.new(x, y).second = y
    finite_set_filter_contains_eq(finite_set_product(range_set(n.suc), range_set(n.suc)),
        square_sum_eq_pred(n), Pair.new(x, y))
    finite_set_filter(finite_set_product(range_set(n.suc), range_set(n.suc)), square_sum_eq_pred(n)).contains(Pair.new(x, y)) =
        (finite_set_product(range_set(n.suc), range_set(n.suc)).contains(Pair.new(x, y))
            and square_sum_eq_pred(n, Pair.new(x, y)))
    finite_set_product_contains_eq(range_set(n.suc), range_set(n.suc), Pair.new(x, y))
    finite_set_product(range_set(n.suc), range_set(n.suc)).contains(Pair.new(x, y)) =
        (range_set(n.suc).contains(x) and range_set(n.suc).contains(y))
    square_sum_eq_pred(n, Pair.new(x, y)) =
        (Pair.new(x, y).first * Pair.new(x, y).first +
            Pair.new(x, y).second * Pair.new(x, y).second = n)
    square_sum_eq_pred(n, Pair.new(x, y)) = (x * x + y * y = n)
    range_set_contains_eq(n.suc, x)
    range_set(n.suc).contains(x) = (x < n.suc)
    range_set_contains_eq(n.suc, y)
    range_set(n.suc).contains(y) = (y < n.suc)
    lt_suc_iff_lte(x, n)
    (x < n.suc) = (x <= n)
    lt_suc_iff_lte(y, n)
    (y < n.suc) = (y <= n)
    if x * x + y * y = n {
        sq_le_imp_le(x * x + y * y, n)
        // x * x <= x * x + y * y
        zero_lte(y * y)
        lte_add_left(x * x, Nat.0, y * y)
        x * x + Nat.0 <= x * x + y * y
        x * x + Nat.0 = x * x
        x * x <= x * x + y * y
        x * x + y * y = n
        x * x + y * y <= n
        lte_trans(x * x, x * x + y * y, n)
        x * x <= n
        sq_le_imp_le(x, n)
        x <= n
        lt_suc_iff_lte(x, n)
        x < n.suc
        sq_le_imp_le(y, n)
        y <= n
        lt_suc_iff_lte(y, n)
        y < n.suc
    }
    (x * x + y * y = n) implies (x < n.suc and y < n.suc)
    ((x < n.suc) and (y < n.suc) and (x * x + y * y = n)) = (x * x + y * y = n)
    two_square_repr(n).contains(Pair.new(x, y)) = (x * x + y * y = n)
}

/// The circle of radius r and the representations of r² are the same set of pairs.
theorem circle_on_points_eq_two_square_repr_sq(r: Nat) {
    circle_on_points(r) = two_square_repr(r * r)
} by {
    forall(p: Pair[Nat, Nat]) {
        pair_eta(p)
        Pair.new(p.first, p.second) = p
        circle_on_points_contains_eq(r, p.first, p.second)
        circle_on_points(r).contains(Pair.new(p.first, p.second)) =
            (p.first * p.first + p.second * p.second = r * r)
        two_square_repr_contains_eq(r * r, p.first, p.second)
        two_square_repr(r * r).contains(Pair.new(p.first, p.second)) =
            (p.first * p.first + p.second * p.second = r * r)
        circle_on_points(r).contains(Pair.new(p.first, p.second)) =
            two_square_repr(r * r).contains(Pair.new(p.first, p.second))
        circle_on_points(r).contains(p) = two_square_repr(r * r).contains(p)
    }
    finite_set_ext_contains(circle_on_points(r), two_square_repr(r * r))
    circle_on_points(r) = two_square_repr(r * r)
}

/// The number of lattice points on the circle x² + y² = r² is the number of
/// ordered representations of r² as a sum of two squares.
theorem circle_on_points_card_eq_two_square_repr(r: Nat) {
    fs_card(circle_on_points(r)) = fs_card(two_square_repr(r * r))
} by {
    circle_on_points_eq_two_square_repr_sq(r)
    circle_on_points(r) = two_square_repr(r * r)
    fs_card(circle_on_points(r)) = fs_card(two_square_repr(r * r))
}

/// The circle x² + y² = r² has a lattice point for every r, namely (r, 0).
theorem circle_on_points_nonempty(r: Nat) {
    exists(x: Nat, y: Nat) {
        circle_on_points(r).contains(Pair.new(x, y))
    }
} by {
    circle_on_points_contains_eq(r, r, Nat.0)
    circle_on_points(r).contains(Pair.new(r, Nat.0)) = (r * r + Nat.0 * Nat.0 = r * r)
    Nat.0 * Nat.0 = Nat.0
    r * r + Nat.0 = r * r
    r * r + Nat.0 * Nat.0 = r * r
    circle_on_points(r).contains(Pair.new(r, Nat.0))
    exists(x: Nat, y: Nat) { circle_on_points(r).contains(Pair.new(x, y)) }
}

/// If p is a prime congruent to 1 modulo 4, the circle x² + y² = p has a lattice
/// point: Fermat's two-squares theorem.
theorem prime_circle_has_lattice_point(p: Nat) {
    p.is_prime and p.mod(Nat.4) = Nat.1 implies exists(x: Nat, y: Nat) {
        x * x + y * y = p
    }
} by {
    if p.is_prime and p.mod(Nat.4) = Nat.1 {
        prime_sum_of_two_squares(p)
        exists(a: Nat, b: Nat) { a * a + b * b = p }
        exists(x: Nat, y: Nat) { x * x + y * y = p }
    }
}

// ============================================================================
// The Gauss circle bound: trivial estimates.
// ============================================================================

/// Predicate: the pair (x, y) lies inside the disk of radius r.
define disk_pred(r: Nat, p: Pair[Nat, Nat]) -> Bool {
    p.first * p.first + p.second * p.second <= r * r
}

/// The lattice points inside the disk x² + y² ≤ r².
define circle_disk_points(r: Nat) -> FiniteSet[Pair[Nat, Nat]] {
    finite_set_filter(finite_set_product(range_set(r.suc), range_set(r.suc)), disk_pred(r))
}

/// A pair (x, y) lies inside the disk x² + y² ≤ r² exactly when x² + y² ≤ r².
theorem circle_disk_points_contains_eq(r: Nat, x: Nat, y: Nat) {
    circle_disk_points(r).contains(Pair.new(x, y)) = (x * x + y * y <= r * r)
} by {
    circle_disk_points(r) =
        finite_set_filter(finite_set_product(range_set(r.suc), range_set(r.suc)), disk_pred(r))
    pair_new_first(x, y)
    Pair.new(x, y).first = x
    pair_new_second(x, y)
    Pair.new(x, y).second = y
    finite_set_filter_contains_eq(finite_set_product(range_set(r.suc), range_set(r.suc)),
        disk_pred(r), Pair.new(x, y))
    finite_set_filter(finite_set_product(range_set(r.suc), range_set(r.suc)), disk_pred(r)).contains(Pair.new(x, y)) =
        (finite_set_product(range_set(r.suc), range_set(r.suc)).contains(Pair.new(x, y))
            and disk_pred(r, Pair.new(x, y)))
    finite_set_product_contains_eq(range_set(r.suc), range_set(r.suc), Pair.new(x, y))
    finite_set_product(range_set(r.suc), range_set(r.suc)).contains(Pair.new(x, y)) =
        (range_set(r.suc).contains(x) and range_set(r.suc).contains(y))
    disk_pred(r, Pair.new(x, y)) =
        (Pair.new(x, y).first * Pair.new(x, y).first +
            Pair.new(x, y).second * Pair.new(x, y).second <= r * r)
    disk_pred(r, Pair.new(x, y)) = (x * x + y * y <= r * r)
    range_set_contains_eq(r.suc, x)
    range_set(r.suc).contains(x) = (x < r.suc)
    range_set_contains_eq(r.suc, y)
    range_set(r.suc).contains(y) = (y < r.suc)
    lt_suc_iff_lte(x, r)
    (x < r.suc) = (x <= r)
    lt_suc_iff_lte(y, r)
    (y < r.suc) = (y <= r)
    if x * x + y * y <= r * r {
        sum_sq_le_imp_first_lte(x, y, r)
        x <= r
        lt_suc_iff_lte(x, r)
        x < r.suc
        sum_sq_le_imp_second_lte(x, y, r)
        y <= r
        lt_suc_iff_lte(y, r)
        y < r.suc
        x < r.suc and y < r.suc
    }
    (x * x + y * y <= r * r) implies (x < r.suc and y < r.suc)
    ((x < r.suc) and (y < r.suc) and (x * x + y * y <= r * r)) =
        (x * x + y * y <= r * r)
    circle_disk_points(r).contains(Pair.new(x, y)) = (x * x + y * y <= r * r)
}

/// Twice the square of the half of n is at most the square of n.
theorem double_half_sq_le(n: Nat) {
    Nat.2 * (n.div(Nat.2) * n.div(Nat.2)) <= n * n
} by {
    div_mod_decomp(n, Nat.2)
    n.div(Nat.2) * Nat.2 + n.mod(Nat.2) = n
    n.div(Nat.2) * Nat.2 <= n
    n.div(Nat.2) * Nat.2 = Nat.2 * n.div(Nat.2)
    Nat.2 * n.div(Nat.2) <= n
    lte_mul_both(n.div(Nat.2), Nat.1, Nat.2)
    n.div(Nat.2) * Nat.1 <= n.div(Nat.2) * Nat.2
    n.div(Nat.2) * Nat.1 = n.div(Nat.2)
    n.div(Nat.2) <= n.div(Nat.2) * Nat.2
    lte_trans(n.div(Nat.2), n.div(Nat.2) * Nat.2, n)
    n.div(Nat.2) <= n
    lte_mul_both(n.div(Nat.2), Nat.2 * n.div(Nat.2), n)
    n.div(Nat.2) * (Nat.2 * n.div(Nat.2)) <= n.div(Nat.2) * n
    lte_mul_right(n, n.div(Nat.2), n)
    n.div(Nat.2) * n <= n * n
    Nat.2 * (n.div(Nat.2) * n.div(Nat.2)) = n.div(Nat.2) * (Nat.2 * n.div(Nat.2))
    lte_trans(Nat.2 * (n.div(Nat.2) * n.div(Nat.2)), n.div(Nat.2) * n, n * n)
    Nat.2 * (n.div(Nat.2) * n.div(Nat.2)) <= n * n
}

/// The disk points lie inside the bounding square, so the count is at most (r+1)².
theorem circle_disk_points_at_most_square(r: Nat) {
    fs_card(circle_disk_points(r)) <= (r + Nat.1) * (r + Nat.1)
} by {
    forall(p: Pair[Nat, Nat]) {
        if circle_disk_points(r).contains(p) {
            pair_eta(p)
            Pair.new(p.first, p.second) = p
            circle_disk_points_contains_eq(r, p.first, p.second)
            circle_disk_points(r).contains(Pair.new(p.first, p.second)) =
                (p.first * p.first + p.second * p.second <= r * r)
            p.first * p.first + p.second * p.second <= r * r
            sum_sq_le_imp_first_lte(p.first, p.second, r)
            p.first <= r
            sum_sq_le_imp_second_lte(p.first, p.second, r)
            p.second <= r
            rectangle_points_contains_eq(r, r, p.first, p.second)
            rectangle_points(r, r).contains(Pair.new(p.first, p.second)) =
                (p.first <= r and p.second <= r)
            rectangle_points(r, r).contains(Pair.new(p.first, p.second))
            rectangle_points(r, r).contains(p)
        }
        circle_disk_points(r).contains(p) implies rectangle_points(r, r).contains(p)
    }
    fs_subset_eq_intro(circle_disk_points(r), rectangle_points(r, r))
    circle_disk_points(r).subset_eq(rectangle_points(r, r))
    rectangle_points_card(r, r)
    fs_card(rectangle_points(r, r)) = (r + Nat.1) * (r + Nat.1)
    fs_card_le_of_subset_le(circle_disk_points(r), rectangle_points(r, r), (r + Nat.1) * (r + Nat.1))
    fs_card(circle_disk_points(r)) <= (r + Nat.1) * (r + Nat.1)
}

/// The quarter square [0, r div 2]² lies inside the disk, so the count is at
/// least (r div 2 + 1)².
theorem circle_disk_points_at_least_quarter_square(r: Nat) {
    (r.div(Nat.2) + Nat.1) * (r.div(Nat.2) + Nat.1) <= fs_card(circle_disk_points(r))
} by {
    forall(p: Pair[Nat, Nat]) {
        if rectangle_points(r.div(Nat.2), r.div(Nat.2)).contains(p) {
            pair_eta(p)
            Pair.new(p.first, p.second) = p
            rectangle_points_contains_eq(r.div(Nat.2), r.div(Nat.2), p.first, p.second)
            rectangle_points(r.div(Nat.2), r.div(Nat.2)).contains(Pair.new(p.first, p.second)) =
                (p.first <= r.div(Nat.2) and p.second <= r.div(Nat.2))
            p.first <= r.div(Nat.2)
            p.second <= r.div(Nat.2)
            square_mono(p.first, r.div(Nat.2))
            p.first * p.first <= r.div(Nat.2) * r.div(Nat.2)
            square_mono(p.second, r.div(Nat.2))
            p.second * p.second <= r.div(Nat.2) * r.div(Nat.2)
            sum_lte(p.first * p.first, p.second * p.second,
                r.div(Nat.2) * r.div(Nat.2), r.div(Nat.2) * r.div(Nat.2))
            p.first * p.first + p.second * p.second <= r.div(Nat.2) * r.div(Nat.2) + r.div(Nat.2) * r.div(Nat.2)
            r.div(Nat.2) * r.div(Nat.2) + r.div(Nat.2) * r.div(Nat.2) =
                Nat.2 * (r.div(Nat.2) * r.div(Nat.2))
            p.first * p.first + p.second * p.second <= Nat.2 * (r.div(Nat.2) * r.div(Nat.2))
            double_half_sq_le(r)
            Nat.2 * (r.div(Nat.2) * r.div(Nat.2)) <= r * r
            lte_trans(p.first * p.first + p.second * p.second,
                Nat.2 * (r.div(Nat.2) * r.div(Nat.2)), r * r)
            p.first * p.first + p.second * p.second <= r * r
            circle_disk_points_contains_eq(r, p.first, p.second)
            circle_disk_points(r).contains(Pair.new(p.first, p.second)) =
                (p.first * p.first + p.second * p.second <= r * r)
            circle_disk_points(r).contains(Pair.new(p.first, p.second))
            circle_disk_points(r).contains(p)
        }
        rectangle_points(r.div(Nat.2), r.div(Nat.2)).contains(p) implies circle_disk_points(r).contains(p)
    }
    fs_subset_eq_intro(rectangle_points(r.div(Nat.2), r.div(Nat.2)), circle_disk_points(r))
    rectangle_points(r.div(Nat.2), r.div(Nat.2)).subset_eq(circle_disk_points(r))
    fs_card_mono(rectangle_points(r.div(Nat.2), r.div(Nat.2)), circle_disk_points(r))
    fs_card(rectangle_points(r.div(Nat.2), r.div(Nat.2))) <= fs_card(circle_disk_points(r))
    rectangle_points_card(r.div(Nat.2), r.div(Nat.2))
    fs_card(rectangle_points(r.div(Nat.2), r.div(Nat.2))) =
        (r.div(Nat.2) + Nat.1) * (r.div(Nat.2) + Nat.1)
    (r.div(Nat.2) + Nat.1) * (r.div(Nat.2) + Nat.1) <= fs_card(circle_disk_points(r))
}

// The classical Gauss circle estimate states that the number N(r) of lattice
// points with x² + y² ≤ r² satisfies N(r) = πr² + O(r). The trivial unit-square
// argument gives |N(r) - πr²| ≤ 4√2·r; both statements need real analysis (the
// area of a disk) and are left for future work. The trivial bounds above,
// (r div 2 + 1)² ≤ N(r) ≤ (r+1)² for the first-quadrant count, are the
// discrete content of that estimate.

// ============================================================================
// The exact triangle count.
// ============================================================================

/// Predicate: the pair (x, y) lies in the triangle x + y ≤ n.
define triangle_pred(n: Nat, p: Pair[Nat, Nat]) -> Bool {
    p.first + p.second <= n
}

/// Predicate: the pair (x, y) lies on the diagonal x + y = n.
define diagonal_pred(n: Nat, p: Pair[Nat, Nat]) -> Bool {
    p.first + p.second = n
}

/// The lattice points in the triangle x + y ≤ n.
define triangle_points(n: Nat) -> FiniteSet[Pair[Nat, Nat]] {
    finite_set_filter(finite_set_product(range_set(n.suc), range_set(n.suc)), triangle_pred(n))
}

/// The lattice points on the diagonal x + y = n.
define diagonal_points(n: Nat) -> FiniteSet[Pair[Nat, Nat]] {
    finite_set_filter(finite_set_product(range_set(n.suc), range_set(n.suc)), diagonal_pred(n))
}

/// A pair (x, y) lies in the triangle x + y ≤ n exactly when x + y ≤ n.
theorem triangle_points_contains_eq(n: Nat, x: Nat, y: Nat) {
    triangle_points(n).contains(Pair.new(x, y)) = (x + y <= n)
} by {
    triangle_points(n) =
        finite_set_filter(finite_set_product(range_set(n.suc), range_set(n.suc)), triangle_pred(n))
    pair_new_first(x, y)
    Pair.new(x, y).first = x
    pair_new_second(x, y)
    Pair.new(x, y).second = y
    finite_set_filter_contains_eq(finite_set_product(range_set(n.suc), range_set(n.suc)),
        triangle_pred(n), Pair.new(x, y))
    finite_set_filter(finite_set_product(range_set(n.suc), range_set(n.suc)), triangle_pred(n)).contains(Pair.new(x, y)) =
        (finite_set_product(range_set(n.suc), range_set(n.suc)).contains(Pair.new(x, y))
            and triangle_pred(n, Pair.new(x, y)))
    finite_set_product_contains_eq(range_set(n.suc), range_set(n.suc), Pair.new(x, y))
    finite_set_product(range_set(n.suc), range_set(n.suc)).contains(Pair.new(x, y)) =
        (range_set(n.suc).contains(x) and range_set(n.suc).contains(y))
    triangle_pred(n, Pair.new(x, y)) =
        (Pair.new(x, y).first + Pair.new(x, y).second <= n)
    triangle_pred(n, Pair.new(x, y)) = (x + y <= n)
    range_set_contains_eq(n.suc, x)
    range_set(n.suc).contains(x) = (x < n.suc)
    range_set_contains_eq(n.suc, y)
    range_set(n.suc).contains(y) = (y < n.suc)
    lt_suc_iff_lte(x, n)
    (x < n.suc) = (x <= n)
    lt_suc_iff_lte(y, n)
    (y < n.suc) = (y <= n)
    if x + y <= n {
        add_lte_imp_left_lte(x, y, n)
        x <= n
        lt_suc_iff_lte(x, n)
        x < n.suc
        add_lte_imp_right_lte(x, y, n)
        y <= n
        lt_suc_iff_lte(y, n)
        y < n.suc
        x < n.suc and y < n.suc
    }
    (x + y <= n) implies (x < n.suc and y < n.suc)
    ((x < n.suc) and (y < n.suc) and (x + y <= n)) = (x + y <= n)
    triangle_points(n).contains(Pair.new(x, y)) = (x + y <= n)
}

/// A pair (x, y) lies on the diagonal x + y = n exactly when x + y = n.
theorem diagonal_points_contains_eq(n: Nat, x: Nat, y: Nat) {
    diagonal_points(n).contains(Pair.new(x, y)) = (x + y = n)
} by {
    diagonal_points(n) =
        finite_set_filter(finite_set_product(range_set(n.suc), range_set(n.suc)), diagonal_pred(n))
    pair_new_first(x, y)
    Pair.new(x, y).first = x
    pair_new_second(x, y)
    Pair.new(x, y).second = y
    finite_set_filter_contains_eq(finite_set_product(range_set(n.suc), range_set(n.suc)),
        diagonal_pred(n), Pair.new(x, y))
    finite_set_filter(finite_set_product(range_set(n.suc), range_set(n.suc)), diagonal_pred(n)).contains(Pair.new(x, y)) =
        (finite_set_product(range_set(n.suc), range_set(n.suc)).contains(Pair.new(x, y))
            and diagonal_pred(n, Pair.new(x, y)))
    finite_set_product_contains_eq(range_set(n.suc), range_set(n.suc), Pair.new(x, y))
    finite_set_product(range_set(n.suc), range_set(n.suc)).contains(Pair.new(x, y)) =
        (range_set(n.suc).contains(x) and range_set(n.suc).contains(y))
    diagonal_pred(n, Pair.new(x, y)) =
        (Pair.new(x, y).first + Pair.new(x, y).second = n)
    diagonal_pred(n, Pair.new(x, y)) = (x + y = n)
    range_set_contains_eq(n.suc, x)
    range_set(n.suc).contains(x) = (x < n.suc)
    range_set_contains_eq(n.suc, y)
    range_set(n.suc).contains(y) = (y < n.suc)
    lt_suc_iff_lte(x, n)
    (x < n.suc) = (x <= n)
    lt_suc_iff_lte(y, n)
    (y < n.suc) = (y <= n)
    if x + y = n {
        add_eq_imp_left_lte(x, y, n)
        x <= n
        lt_suc_iff_lte(x, n)
        x < n.suc
        add_eq_imp_right_lte(x, y, n)
        y <= n
        lt_suc_iff_lte(y, n)
        y < n.suc
        x < n.suc and y < n.suc
    }
    (x + y = n) implies (x < n.suc and y < n.suc)
    ((x < n.suc) and (y < n.suc) and (x + y = n)) = (x + y = n)
    diagonal_points(n).contains(Pair.new(x, y)) = (x + y = n)
}

/// The pair (x, n - x) on the diagonal x + y = n.
define diagonal_fn(n: Nat, x: Nat) -> Pair[Nat, Nat] {
    Pair.new(x, n - x)
}

/// The list of pairs (x, n - x) for x < n.suc.
define diagonal_list(n: Nat) -> List[Pair[Nat, Nat]] {
    map[Nat, Pair[Nat, Nat]](n.suc.range, diagonal_fn(n))
}

/// The diagonal list has length n + 1.
theorem diagonal_list_length(n: Nat) {
    diagonal_list(n).length = n.suc
} by {
    map_length(n.suc.range, diagonal_fn(n))
    map[Nat, Pair[Nat, Nat]](n.suc.range, diagonal_fn(n)).length = n.suc.range.length
    length_range(n.suc)
    n.suc.range.length = n.suc
    diagonal_list(n).length = n.suc
}

/// Pairing x with n - x is injective in x.
theorem diagonal_fn_injective(n: Nat) {
    is_injective_fn(diagonal_fn(n))
} by {
    forall(x1: Nat, x2: Nat) {
        if diagonal_fn(n, x1) = diagonal_fn(n, x2) {
            diagonal_fn(n, x1) = Pair.new(x1, n - x1)
            diagonal_fn(n, x2) = Pair.new(x2, n - x2)
            Pair.new(x1, n - x1) = Pair.new(x2, n - x2)
            (Pair.new(x1, n - x1)).first = (Pair.new(x2, n - x2)).first
            pair_new_first(x1, n - x1)
            Pair.new(x1, n - x1).first = x1
            pair_new_first(x2, n - x2)
            Pair.new(x2, n - x2).first = x2
            x1 = x2
        }
    }
}

/// The diagonal list has no repeated pairs.
theorem diagonal_list_unique(n: Nat) {
    diagonal_list(n).is_unique
} by {
    range_is_unique(n.suc)
    n.suc.range.is_unique
    diagonal_fn_injective(n)
    is_injective_fn(diagonal_fn(n))
    injective_map_is_unique[Nat, Pair[Nat, Nat]](n.suc.range, diagonal_fn(n))
    map[Nat, Pair[Nat, Nat]](n.suc.range, diagonal_fn(n)).is_unique
    diagonal_list(n).is_unique
}

/// A pair (x, y) occurs in the diagonal list exactly when x + y = n.
theorem diagonal_list_membership(n: Nat, x: Nat, y: Nat) {
    diagonal_list(n).contains(Pair.new(x, y)) = (x + y = n)
} by {
    if diagonal_list(n).contains(Pair.new(x, y)) {
        diagonal_list(n) = map[Nat, Pair[Nat, Nat]](n.suc.range, diagonal_fn(n))
        map_contains(n.suc.range, diagonal_fn(n), Pair.new(x, y))
        exists(z: Nat) { n.suc.range.contains(z) and diagonal_fn(n, z) = Pair.new(x, y) }
        let z: Nat satisfy { n.suc.range.contains(z) and diagonal_fn(n, z) = Pair.new(x, y) }
        lt_of_range_contains(n.suc, z)
        z < n.suc
        lt_suc_iff_lte(z, n)
        z <= n
        diagonal_fn(n, z) = Pair.new(z, n - z)
        Pair.new(z, n - z) = Pair.new(x, y)
        (Pair.new(z, n - z)).first = (Pair.new(x, y)).first
        pair_new_first(z, n - z)
        Pair.new(z, n - z).first = z
        pair_new_first(x, y)
        Pair.new(x, y).first = x
        z = x
        (Pair.new(z, n - z)).second = (Pair.new(x, y)).second
        pair_new_second(z, n - z)
        Pair.new(z, n - z).second = n - z
        pair_new_second(x, y)
        Pair.new(x, y).second = y
        n - z = y
        add_sub(n, z)
        (n - z) + z = n
        y + z = n
        y + x = n
        x + y = n
    }
    (diagonal_list(n).contains(Pair.new(x, y))) implies (x + y = n)
    if x + y = n {
        add_eq_imp_left_lte(x, y, n)
        x <= n
        lt_suc_iff_lte(x, n)
        x < n.suc
        range_contains_of_lt(n.suc, x)
        n.suc.range.contains(x)
        add_sub(n, x)
        (n - x) + x = n
        add_comm(x, y)
        x + y = y + x
        y + x = n
        add_cancels_right(x, n - x, y)
        n - x = y
        diagonal_fn(n, x) = Pair.new(x, n - x)
        diagonal_fn(n, x) = Pair.new(x, y)
        map_contains_of_contains(n.suc.range, diagonal_fn(n), x)
        map[Nat, Pair[Nat, Nat]](n.suc.range, diagonal_fn(n)).contains(diagonal_fn(n, x))
        diagonal_list(n).contains(diagonal_fn(n, x))
        diagonal_list(n).contains(Pair.new(x, y))
    }
    (x + y = n) implies (diagonal_list(n).contains(Pair.new(x, y)))
    diagonal_list(n).contains(Pair.new(x, y)) = (x + y = n)
}

/// The diagonal set is represented by the diagonal list.
theorem diagonal_points_eq_fs_from_list(n: Nat) {
    diagonal_points(n) = fs_from_list(diagonal_list(n))
} by {
    forall(p: Pair[Nat, Nat]) {
        pair_eta(p)
        Pair.new(p.first, p.second) = p
        diagonal_points_contains_eq(n, p.first, p.second)
        diagonal_points(n).contains(Pair.new(p.first, p.second)) = (p.first + p.second = n)
        diagonal_list_membership(n, p.first, p.second)
        diagonal_list(n).contains(Pair.new(p.first, p.second)) = (p.first + p.second = n)
        fs_from_list_contains_eq(diagonal_list(n), Pair.new(p.first, p.second))
        fs_from_list(diagonal_list(n)).contains(Pair.new(p.first, p.second)) =
            diagonal_list(n).contains(Pair.new(p.first, p.second))
        fs_from_list(diagonal_list(n)).contains(Pair.new(p.first, p.second)) = (p.first + p.second = n)
        diagonal_points(n).contains(Pair.new(p.first, p.second)) =
            fs_from_list(diagonal_list(n)).contains(Pair.new(p.first, p.second))
        diagonal_points(n).contains(p) = fs_from_list(diagonal_list(n)).contains(p)
    }
    finite_set_ext_contains(diagonal_points(n), fs_from_list(diagonal_list(n)))
    diagonal_points(n) = fs_from_list(diagonal_list(n))
}

/// The diagonal x + y = n has exactly n + 1 lattice points.
theorem diagonal_points_card(n: Nat) {
    fs_card(diagonal_points(n)) = n + Nat.1
} by {
    diagonal_points_eq_fs_from_list(n)
    diagonal_points(n) = fs_from_list(diagonal_list(n))
    diagonal_list_unique(n)
    diagonal_list(n).is_unique
    finite_set_from_unique_list_cardinality_is_length(diagonal_list(n))
    fs_from_list(diagonal_list(n)).cardinality_is(diagonal_list(n).length)
    diagonal_list_length(n)
    diagonal_list(n).length = n.suc
    fs_from_list(diagonal_list(n)).cardinality_is(n.suc)
    diagonal_points(n).cardinality_is(n.suc)
    fs_card_eq_of_cardinality_is(diagonal_points(n), n.suc)
    fs_card(diagonal_points(n)) = n.suc
}

/// A pair built from components equals the zero pair exactly when both components are zero.
theorem pair_new_zero_eq_iff_components(x: Nat, y: Nat) {
    (Pair.new(x, y) = Pair.new(Nat.0, Nat.0)) = (x = Nat.0 and y = Nat.0)
} by {
    if Pair.new(x, y) = Pair.new(Nat.0, Nat.0) {
        (Pair.new(x, y)).first = (Pair.new(Nat.0, Nat.0)).first
        pair_new_first(x, y)
        Pair.new(x, y).first = x
        pair_new_first(Nat.0, Nat.0)
        Pair.new(Nat.0, Nat.0).first = Nat.0
        x = Nat.0
        (Pair.new(x, y)).second = (Pair.new(Nat.0, Nat.0)).second
        pair_new_second(x, y)
        Pair.new(x, y).second = y
        pair_new_second(Nat.0, Nat.0)
        Pair.new(Nat.0, Nat.0).second = Nat.0
        y = Nat.0
        x = Nat.0 and y = Nat.0
    }
    (Pair.new(x, y) = Pair.new(Nat.0, Nat.0)) implies (x = Nat.0 and y = Nat.0)
    if x = Nat.0 and y = Nat.0 {
        x = Nat.0
        y = Nat.0
        Pair.new(x, y) = Pair.new(Nat.0, Nat.0)
    }
    (x = Nat.0 and y = Nat.0) implies (Pair.new(x, y) = Pair.new(Nat.0, Nat.0))
    (Pair.new(x, y) = Pair.new(Nat.0, Nat.0)) = (x = Nat.0 and y = Nat.0)
}

/// The triangle of side zero is the singleton {(0, 0)}.
theorem triangle_points_zero_eq_singleton {
    triangle_points(Nat.0) = fs_insert(FiniteSet.empty[Pair[Nat, Nat]], Pair.new(Nat.0, Nat.0))
} by {
    forall(p: Pair[Nat, Nat]) {
        pair_eta(p)
        Pair.new(p.first, p.second) = p
        triangle_points_contains_eq(Nat.0, p.first, p.second)
        triangle_points(Nat.0).contains(Pair.new(p.first, p.second)) = (p.first + p.second <= Nat.0)
        if p.first + p.second <= Nat.0 {
            lte_zero_imp_zero(p.first + p.second)
            p.first + p.second = Nat.0
            add_to_zero(p.first, p.second)
            p.first = Nat.0 and p.second = Nat.0
        }
        if p.first = Nat.0 and p.second = Nat.0 {
            p.first = Nat.0
            p.second = Nat.0
            p.first + p.second = Nat.0
            lte_ref(Nat.0)
            p.first + p.second <= Nat.0
        }
        (p.first + p.second <= Nat.0) = (p.first = Nat.0 and p.second = Nat.0)
        triangle_points(Nat.0).contains(Pair.new(p.first, p.second)) = (p.first = Nat.0 and p.second = Nat.0)
        pair_new_zero_eq_iff_components(p.first, p.second)
        (Pair.new(p.first, p.second) = Pair.new(Nat.0, Nat.0)) = (p.first = Nat.0 and p.second = Nat.0)
        finite_set_singleton_contains_eq(Pair.new(Nat.0, Nat.0), Pair.new(p.first, p.second))
        fs_insert(FiniteSet.empty[Pair[Nat, Nat]], Pair.new(Nat.0, Nat.0)).contains(Pair.new(p.first, p.second)) =
            (Pair.new(p.first, p.second) = Pair.new(Nat.0, Nat.0))
        fs_insert(FiniteSet.empty[Pair[Nat, Nat]], Pair.new(Nat.0, Nat.0)).contains(Pair.new(p.first, p.second)) =
            (p.first = Nat.0 and p.second = Nat.0)
        triangle_points(Nat.0).contains(Pair.new(p.first, p.second)) =
            fs_insert(FiniteSet.empty[Pair[Nat, Nat]], Pair.new(Nat.0, Nat.0)).contains(Pair.new(p.first, p.second))
        triangle_points(Nat.0).contains(p) =
            fs_insert(FiniteSet.empty[Pair[Nat, Nat]], Pair.new(Nat.0, Nat.0)).contains(p)
    }
    finite_set_ext_contains(triangle_points(Nat.0),
        fs_insert(FiniteSet.empty[Pair[Nat, Nat]], Pair.new(Nat.0, Nat.0)))
    triangle_points(Nat.0) = fs_insert(FiniteSet.empty[Pair[Nat, Nat]], Pair.new(Nat.0, Nat.0))
}

/// The triangle of side n + 1 is the triangle of side n together with the
/// diagonal x + y = n + 1.
theorem triangle_suc_union(n: Nat) {
    triangle_points(n.suc) = fs_union(triangle_points(n), diagonal_points(n.suc))
} by {
    forall(p: Pair[Nat, Nat]) {
        pair_eta(p)
        Pair.new(p.first, p.second) = p
        triangle_points_contains_eq(n.suc, p.first, p.second)
        triangle_points(n.suc).contains(Pair.new(p.first, p.second)) = (p.first + p.second <= n.suc)
        triangle_points_contains_eq(n, p.first, p.second)
        triangle_points(n).contains(Pair.new(p.first, p.second)) = (p.first + p.second <= n)
        diagonal_points_contains_eq(n.suc, p.first, p.second)
        diagonal_points(n.suc).contains(Pair.new(p.first, p.second)) = (p.first + p.second = n.suc)
        lte_suc_or_eq(p.first + p.second, n)
        (p.first + p.second <= n.suc) = (p.first + p.second <= n or p.first + p.second = n.suc)
        finite_set_union_contains_eq(triangle_points(n), diagonal_points(n.suc), Pair.new(p.first, p.second))
        fs_union(triangle_points(n), diagonal_points(n.suc)).contains(Pair.new(p.first, p.second)) =
            (triangle_points(n).contains(Pair.new(p.first, p.second)) or
                diagonal_points(n.suc).contains(Pair.new(p.first, p.second)))
        triangle_points(n.suc).contains(Pair.new(p.first, p.second)) =
            fs_union(triangle_points(n), diagonal_points(n.suc)).contains(Pair.new(p.first, p.second))
        triangle_points(n.suc).contains(p) = fs_union(triangle_points(n), diagonal_points(n.suc)).contains(p)
    }
    finite_set_ext_contains(triangle_points(n.suc), fs_union(triangle_points(n), diagonal_points(n.suc)))
    triangle_points(n.suc) = fs_union(triangle_points(n), diagonal_points(n.suc))
}

/// The triangle of side n is disjoint from the diagonal x + y = n + 1.
theorem triangle_diagonal_disjoint(n: Nat) {
    triangle_points(n).is_disjoint(diagonal_points(n.suc))
} by {
    triangle_points(n).is_disjoint(diagonal_points(n.suc)) =
        triangle_points(n).underlying_set.is_disjoint(diagonal_points(n.suc).underlying_set)
    triangle_points(n).underlying_set.is_disjoint(diagonal_points(n.suc).underlying_set) =
        forall(p: Pair[Nat, Nat]) {
            not (triangle_points(n).underlying_set.contains(p) and
                diagonal_points(n.suc).underlying_set.contains(p))
        }
    forall(p: Pair[Nat, Nat]) {
        triangle_points(n).underlying_set.contains(p) = triangle_points(n).contains(p)
        diagonal_points(n.suc).underlying_set.contains(p) = diagonal_points(n.suc).contains(p)
        pair_eta(p)
        Pair.new(p.first, p.second) = p
        if triangle_points(n).contains(p) and diagonal_points(n.suc).contains(p) {
            triangle_points_contains_eq(n, p.first, p.second)
            triangle_points(n).contains(Pair.new(p.first, p.second)) = (p.first + p.second <= n)
            p.first + p.second <= n
            diagonal_points_contains_eq(n.suc, p.first, p.second)
            diagonal_points(n.suc).contains(Pair.new(p.first, p.second)) = (p.first + p.second = n.suc)
            p.first + p.second = n.suc
            lte_ref(n.suc)
            n.suc <= n.suc
            n.suc <= p.first + p.second
            lte_trans(n.suc, p.first + p.second, n)
            n.suc <= n
            lt_suc(n)
            n < n.suc
            lt_imp_lte(n, n.suc)
            n <= n.suc
            lte_antisymm(n.suc, n)
            n.suc = n
            suc_ne(n)
            n.suc != n
            false
        }
        not (triangle_points(n).contains(p) and diagonal_points(n.suc).contains(p))
        not (triangle_points(n).underlying_set.contains(p) and diagonal_points(n.suc).underlying_set.contains(p))
    }
    triangle_points(n).underlying_set.is_disjoint(diagonal_points(n.suc).underlying_set)
    triangle_points(n).is_disjoint(diagonal_points(n.suc))
}

/// The triangle of side n has triangular(n + 1) lattice points.
theorem triangle_points_card_eq_triangular(n: Nat) {
    fs_card(triangle_points(n)) = triangular(n.suc)
} by {
    define p(m: Nat) -> Bool {
        fs_card(triangle_points(m)) = triangular(m.suc)
    }
    triangle_points_zero_eq_singleton
    triangle_points(Nat.0) = fs_insert(FiniteSet.empty[Pair[Nat, Nat]], Pair.new(Nat.0, Nat.0))
    fs_card_singleton(Pair.new(Nat.0, Nat.0))
    fs_card(FiniteSet.empty[Pair[Nat, Nat]].insert(Pair.new(Nat.0, Nat.0))) = Nat.1
    fs_card(triangle_points(Nat.0)) = Nat.1
    triangular_zero
    triangular(Nat.0) = Nat.0
    triangular_suc(Nat.0)
    triangular(Nat.1) = Nat.1 + triangular(Nat.0)
    triangular(Nat.1) = Nat.1
    fs_card(triangle_points(Nat.0)) = triangular(Nat.1)
    p(Nat.0)
    forall(k: Nat) {
        if p(k) {
            fs_card(triangle_points(k)) = triangular(k.suc)
            triangle_suc_union(k)
            triangle_points(k.suc) = fs_union(triangle_points(k), diagonal_points(k.suc))
            triangle_diagonal_disjoint(k)
            triangle_points(k).is_disjoint(diagonal_points(k.suc))
            fs_card_cardinality_is(triangle_points(k))
            triangle_points(k).cardinality_is(fs_card(triangle_points(k)))
            fs_card_cardinality_is(diagonal_points(k.suc))
            diagonal_points(k.suc).cardinality_is(fs_card(diagonal_points(k.suc)))
            finite_set_disjoint_union_cardinality_is(triangle_points(k), diagonal_points(k.suc),
                fs_card(triangle_points(k)), fs_card(diagonal_points(k.suc)))
            fs_union(triangle_points(k), diagonal_points(k.suc)).cardinality_is(
                fs_card(triangle_points(k)) + fs_card(diagonal_points(k.suc)))
            triangle_points(k.suc).cardinality_is(
                fs_card(triangle_points(k)) + fs_card(diagonal_points(k.suc)))
            fs_card_eq_of_cardinality_is(triangle_points(k.suc),
                fs_card(triangle_points(k)) + fs_card(diagonal_points(k.suc)))
            fs_card(triangle_points(k.suc)) = fs_card(triangle_points(k)) + fs_card(diagonal_points(k.suc))
            fs_card(triangle_points(k)) = triangular(k.suc)
            diagonal_points_card(k.suc)
            fs_card(diagonal_points(k.suc)) = k.suc.suc
            fs_card(triangle_points(k.suc)) = triangular(k.suc) + k.suc.suc
            triangular_suc(k.suc)
            triangular(k.suc.suc) = k.suc.suc + triangular(k.suc)
            add_comm(triangular(k.suc), k.suc.suc)
            triangular(k.suc) + k.suc.suc = k.suc.suc + triangular(k.suc)
            fs_card(triangle_points(k.suc)) = triangular(k.suc.suc)
            p(k.suc)
        }
        (p(k) implies p(k.suc))
    }
    p(Nat.0) and forall(k: Nat) { p(k) implies p(k.suc) }
    Nat.induction(p)
    p(n)
    fs_card(triangle_points(n)) = triangular(n.suc)
}

/// Twice the number of lattice points in the triangle x + y ≤ n is (n+1)(n+2).
theorem triangle_points_card(n: Nat) {
    Nat.2 * fs_card(triangle_points(n)) = (n + Nat.1) * (n + Nat.2)
} by {
    triangle_points_card_eq_triangular(n)
    fs_card(triangle_points(n)) = triangular(n.suc)
    triangular_doubled(n.suc)
    Nat.2 * triangular(n.suc) = n.suc * (n.suc + Nat.1)
    Nat.2 * fs_card(triangle_points(n)) = n.suc * (n.suc + Nat.1)
    Nat.2 * fs_card(triangle_points(n)) = (n + Nat.1) * (n + Nat.2)
}

/// The number of lattice points in the triangle x + y ≤ n is (n+1)(n+2)/2.
theorem triangle_points_card_half(n: Nat) {
    fs_card(triangle_points(n)) = ((n + Nat.1) * (n + Nat.2)).div(Nat.2)
} by {
    triangle_points_card(n)
    Nat.2 * fs_card(triangle_points(n)) = (n + Nat.1) * (n + Nat.2)
    mul_comm(Nat.2, fs_card(triangle_points(n)))
    Nat.2 * fs_card(triangle_points(n)) = fs_card(triangle_points(n)) * Nat.2
    fs_card(triangle_points(n)) * Nat.2 = (n + Nat.1) * (n + Nat.2)
    div_mul(fs_card(triangle_points(n)), Nat.2)
    (fs_card(triangle_points(n)) * Nat.2).div(Nat.2) = fs_card(triangle_points(n))
    ((n + Nat.1) * (n + Nat.2)).div(Nat.2) = fs_card(triangle_points(n))
    fs_card(triangle_points(n)) = ((n + Nat.1) * (n + Nat.2)).div(Nat.2)
}

// ============================================================================
// The signed rectangle: |x| ≤ m and |y| ≤ n.
// ============================================================================

/// The integer i - m, indexing the integers from -m upward.
define int_index(m: Nat, i: Nat) -> Int {
    sub_nat(i, m)
}

/// The integers from -m to m, in increasing order, as a list.
define int_range(m: Nat) -> List[Int] {
    map[Nat, Int]((Nat.2 * m + Nat.1).range, int_index(m))
}

/// The signed range list has length 2m + 1.
theorem int_range_length(m: Nat) {
    int_range(m).length = Nat.2 * m + Nat.1
} by {
    map_length((Nat.2 * m + Nat.1).range, int_index(m))
    map[Nat, Int]((Nat.2 * m + Nat.1).range, int_index(m)).length =
        (Nat.2 * m + Nat.1).range.length
    length_range(Nat.2 * m + Nat.1)
    (Nat.2 * m + Nat.1).range.length = Nat.2 * m + Nat.1
    int_range(m).length = Nat.2 * m + Nat.1
}

/// Indexing is injective: distinct indices give distinct integers.
theorem int_index_injective(m: Nat) {
    is_injective_fn(int_index(m))
} by {
    forall(i: Nat, j: Nat) {
        if int_index(m, i) = int_index(m, j) {
            int_index(m, i) = sub_nat(i, m)
            int_index(m, j) = sub_nat(j, m)
            sub_nat(i, m) = sub_nat(j, m)
            sub_nat_cancel_right(i, j, m)
            i = j
        }
    }
}

/// The signed range list has no repeated integers.
theorem int_range_unique(m: Nat) {
    int_range(m).is_unique
} by {
    range_is_unique(Nat.2 * m + Nat.1)
    (Nat.2 * m + Nat.1).range.is_unique
    int_index_injective(m)
    is_injective_fn(int_index(m))
    injective_map_is_unique[Nat, Int]((Nat.2 * m + Nat.1).range, int_index(m))
    map[Nat, Int]((Nat.2 * m + Nat.1).range, int_index(m)).is_unique
    int_range(m).is_unique
}

/// The finite set of integers from -m to m.
define signed_range(m: Nat) -> FiniteSet[Int] {
    fs_from_list(int_range(m))
}

/// The signed range [-m, m] has exactly 2m + 1 integers.
theorem signed_range_card(m: Nat) {
    fs_card(signed_range(m)) = Nat.2 * m + Nat.1
} by {
    signed_range(m) = fs_from_list(int_range(m))
    int_range_unique(m)
    int_range(m).is_unique
    finite_set_from_unique_list_cardinality_is_length(int_range(m))
    fs_from_list(int_range(m)).cardinality_is(int_range(m).length)
    int_range_length(m)
    int_range(m).length = Nat.2 * m + Nat.1
    fs_from_list(int_range(m)).cardinality_is(Nat.2 * m + Nat.1)
    signed_range(m).cardinality_is(Nat.2 * m + Nat.1)
    fs_card_eq_of_cardinality_is(signed_range(m), Nat.2 * m + Nat.1)
    fs_card(signed_range(m)) = Nat.2 * m + Nat.1
}

/// The rectangle of integer lattice points with |x| ≤ m and |y| ≤ n.
define signed_rectangle(m: Nat, n: Nat) -> FiniteSet[Pair[Int, Int]] {
    finite_set_product(signed_range(m), signed_range(n))
}

/// The number of integer lattice points (x, y) with |x| ≤ m and |y| ≤ n is (2m+1)(2n+1).
theorem signed_rectangle_card(m: Nat, n: Nat) {
    fs_card(signed_rectangle(m, n)) = (Nat.2 * m + Nat.1) * (Nat.2 * n + Nat.1)
} by {
    signed_rectangle(m, n) = finite_set_product(signed_range(m), signed_range(n))
    signed_range_card(m)
    fs_card(signed_range(m)) = Nat.2 * m + Nat.1
    signed_range_card(n)
    fs_card(signed_range(n)) = Nat.2 * n + Nat.1
    fs_card_product(signed_range(m), signed_range(n))
    fs_card(finite_set_product(signed_range(m), signed_range(n))) =
        fs_card(signed_range(n)) * fs_card(signed_range(m))
    fs_card(signed_rectangle(m, n)) = (Nat.2 * n + Nat.1) * (Nat.2 * m + Nat.1)
    fs_card(signed_rectangle(m, n)) = (Nat.2 * m + Nat.1) * (Nat.2 * n + Nat.1)
}

// ============================================================================
// Natural-number subtraction lemmas for the signed range.
// ============================================================================

/// Subtraction does not increase a natural.
theorem sub_le(a: Nat, b: Nat) {
    a - b <= a
} by {
    lt_or_lte(a, b)
    a < b or b <= a
    if a < b {
        sub_lt(a, b)
        a - b = Nat.0
        zero_lte(a)
        Nat.0 <= a
        a - b <= a
    }
    if b <= a {
        add_sub(a, b)
        (a - b) + b = a
        a - b <= a
    }
    a - b <= a
}

/// Subtracting a positive amount strictly decreases a natural.
theorem sub_lt_self(a: Nat, b: Nat) {
    b <= a and b != Nat.0 implies a - b < a
} by {
    if b <= a and b != Nat.0 {
        sub_le(a, b)
        a - b <= a
        if a - b = a {
            add_sub(a, b)
            (a - b) + b = a
            a + b = a
            add_identity_right(a, b)
            b = Nat.0
            false
        }
        a - b != a
        lte_neq_imp_lt(a - b, a)
        a - b < a
    }
}

/// Subtracting twice undoes subtraction: a - (a - b) = b for b ≤ a.
theorem sub_sub(a: Nat, b: Nat) {
    b <= a implies a - (a - b) = b
} by {
    if b <= a {
        add_sub(a, b)
        (a - b) + b = a
        add_comm(a - b, b)
        (a - b) + b = b + (a - b)
        b + (a - b) = a
        add_imp_sub(b, a - b, a)
        a - (a - b) = b
    }
}

/// Subtraction is monotone in its left argument.
theorem lte_sub_right(a: Nat, b: Nat, c: Nat) {
    a <= b implies a - c <= b - c
} by {
    if a <= b {
        if a < c {
            sub_lt(a, c)
            a - c = Nat.0
            zero_lte(b - c)
            Nat.0 <= b - c
            a - c <= b - c
        }
        if not a < c {
            c <= a
            add_sub(a, c)
            (a - c) + c = a
            lte_trans(c, a, b)
            c <= b
            add_sub(b, c)
            (b - c) + c = b
            let d: Nat satisfy { a + d = b }
            a + d = b
            (b - c) + c = a + d
            ((a - c) + c) + d = (a - c) + (c + d)
            a + d = ((a - c) + c) + d
            (b - c) + c = ((a - c) + c) + d
            ((a - c) + c) + d = ((a - c) + d) + c
            (b - c) + c = ((a - c) + d) + c
            add_cancels_right(c, b - c, (a - c) + d)
            b - c = (a - c) + d
            a - c <= b - c
        }
        a - c <= b - c
    }
}

// ============================================================================
// The signed range is exactly the integers with small absolute value.
// ============================================================================

/// A member of the signed range has absolute value at most m.
theorem signed_range_contains_imp_abs(m: Nat, z: Int) {
    signed_range(m).contains(z) implies abs(z) <= m
} by {
    if signed_range(m).contains(z) {
        signed_range(m) = fs_from_list(int_range(m))
        fs_from_list_contains_eq(int_range(m), z)
        fs_from_list(int_range(m)).contains(z) = int_range(m).contains(z)
        int_range(m).contains(z)
        int_range(m) = map[Nat, Int]((Nat.2 * m + Nat.1).range, int_index(m))
        map_contains((Nat.2 * m + Nat.1).range, int_index(m), z)
        exists(i: Nat) {
            (Nat.2 * m + Nat.1).range.contains(i) and int_index(m, i) = z
        }
        let i: Nat satisfy {
            (Nat.2 * m + Nat.1).range.contains(i) and int_index(m, i) = z
        }
        lt_of_range_contains(Nat.2 * m + Nat.1, i)
        i < Nat.2 * m + Nat.1
        int_index(m, i) = sub_nat(i, m)
        sub_nat(i, m) = z
        abs(sub_nat(i, m)) = abs(z)
        Nat.2 * m + Nat.1 = (Nat.2 * m).suc
        i < (Nat.2 * m).suc
        lt_suc_iff_lte(i, Nat.2 * m)
        i <= Nat.2 * m
        sub_nat(i, m) = if m <= i { Int.from_nat(i - m) } else { -(Int.from_nat(m - i)) }
        if m <= i {
            sub_nat(i, m) = Int.from_nat(i - m)
            abs(Int.from_nat(i - m)) = abs(z)
            abs_from_nat(i - m)
            abs(Int.from_nat(i - m)) = i - m
            i - m = abs(z)
            lte_sub_right(i, Nat.2 * m, m)
            i - m <= Nat.2 * m - m
            mul_two_left(m)
            Nat.2 * m = m + m
            add_imp_sub(m, m, m + m)
            (m + m) - m = m
            Nat.2 * m - m = m
            i - m <= m
            abs(z) <= m
        }
        if not m <= i {
            lt_or_lte(i, m)
            i < m or m <= i
            not m <= i
            i < m
            sub_nat(i, m) = -(Int.from_nat(m - i))
            abs(-(Int.from_nat(m - i))) = abs(z)
            abs_neg(Int.from_nat(m - i))
            abs(-(Int.from_nat(m - i))) = abs(Int.from_nat(m - i))
            abs_from_nat(m - i)
            abs(Int.from_nat(m - i)) = m - i
            m - i = abs(z)
            sub_le(m, i)
            m - i <= m
            abs(z) <= m
        }
        abs(z) <= m
    }
}

/// An integer with absolute value at most m lies in the signed range.
theorem signed_range_abs_imp_contains(m: Nat, z: Int) {
    abs(z) <= m implies signed_range(m).contains(z)
} by {
    if abs(z) <= m {
        neg_or_pos(z)
        z = Int.from_nat(abs(z)) or z = -(Int.from_nat(abs(z)))
        if z = Int.from_nat(abs(z)) {
            int_index(m, m + abs(z)) = sub_nat(m + abs(z), m)
            sub_nat_add_left(abs(z), m)
            sub_nat(abs(z) + m, m) = Int.from_nat(abs(z))
            add_comm(abs(z), m)
            abs(z) + m = m + abs(z)
            sub_nat(m + abs(z), m) = Int.from_nat(abs(z))
            int_index(m, m + abs(z)) = Int.from_nat(abs(z))
            int_index(m, m + abs(z)) = z
            lte_add_right(m, abs(z), m)
            abs(z) + m <= m + m
            m + abs(z) <= m + m
            mul_two_left(m)
            Nat.2 * m = m + m
            m + abs(z) <= Nat.2 * m
            lt_suc(Nat.2 * m)
            Nat.2 * m < (Nat.2 * m).suc
            Nat.2 * m + Nat.1 = (Nat.2 * m).suc
            Nat.2 * m < Nat.2 * m + Nat.1
            lte_and_lt(m + abs(z), Nat.2 * m, Nat.2 * m + Nat.1)
            m + abs(z) < Nat.2 * m + Nat.1
            range_contains_of_lt(Nat.2 * m + Nat.1, m + abs(z))
            (Nat.2 * m + Nat.1).range.contains(m + abs(z))
            map_contains_of_contains((Nat.2 * m + Nat.1).range, int_index(m), m + abs(z))
            map[Nat, Int]((Nat.2 * m + Nat.1).range, int_index(m)).contains(int_index(m, m + abs(z)))
            int_range(m).contains(int_index(m, m + abs(z)))
            int_range(m).contains(z)
        }
        if z = -(Int.from_nat(abs(z))) {
            if abs(z) = Nat.0 {
                add_imp_sub(m, Nat.0, m)
                m - Nat.0 = m
                m - abs(z) = m
                sub_nat_self(m)
                sub_nat(m, m) = Int.from_nat(Nat.0)
                sub_nat(m - abs(z), m) = Int.from_nat(Nat.0)
                neg_zero
                -Int.from_nat(Nat.0) = Int.from_nat(Nat.0)
                z = -(Int.from_nat(abs(z)))
                z = -(Int.from_nat(Nat.0))
                z = Int.from_nat(Nat.0)
                sub_nat(m - abs(z), m) = z
                int_index(m, m - abs(z)) = sub_nat(m - abs(z), m)
                int_index(m, m - abs(z)) = z
                sub_le(m, abs(z))
                m - abs(z) <= m
                lt_add_suc(m, m)
                m < m + m.suc
                m + m.suc = Nat.2 * m + Nat.1
                m < Nat.2 * m + Nat.1
                lte_and_lt(m - abs(z), m, Nat.2 * m + Nat.1)
                m - abs(z) < Nat.2 * m + Nat.1
                range_contains_of_lt(Nat.2 * m + Nat.1, m - abs(z))
                (Nat.2 * m + Nat.1).range.contains(m - abs(z))
                map_contains_of_contains((Nat.2 * m + Nat.1).range, int_index(m), m - abs(z))
                map[Nat, Int]((Nat.2 * m + Nat.1).range, int_index(m)).contains(int_index(m, m - abs(z)))
                int_range(m).contains(int_index(m, m - abs(z)))
                int_range(m).contains(z)
            }
            if abs(z) != Nat.0 {
                sub_lt_self(m, abs(z))
                m - abs(z) < m
                sub_nat(m - abs(z), m) = if m <= m - abs(z) {
                    Int.from_nat((m - abs(z)) - m)
                } else {
                    -(Int.from_nat(m - (m - abs(z))))
                }
                if m <= m - abs(z) {
                    lte_and_lt(m, m - abs(z), m)
                    m < m
                    lt_not_ref(m)
                    false
                }
                not m <= m - abs(z)
                sub_nat(m - abs(z), m) = -(Int.from_nat(m - (m - abs(z))))
                sub_sub(m, abs(z))
                m - (m - abs(z)) = abs(z)
                sub_nat(m - abs(z), m) = -(Int.from_nat(abs(z)))
                z = -(Int.from_nat(abs(z)))
                sub_nat(m - abs(z), m) = z
                int_index(m, m - abs(z)) = sub_nat(m - abs(z), m)
                int_index(m, m - abs(z)) = z
                sub_le(m, abs(z))
                m - abs(z) <= m
                lt_add_suc(m, m)
                m < m + m.suc
                m + m.suc = Nat.2 * m + Nat.1
                m < Nat.2 * m + Nat.1
                lte_and_lt(m - abs(z), m, Nat.2 * m + Nat.1)
                m - abs(z) < Nat.2 * m + Nat.1
                range_contains_of_lt(Nat.2 * m + Nat.1, m - abs(z))
                (Nat.2 * m + Nat.1).range.contains(m - abs(z))
                map_contains_of_contains((Nat.2 * m + Nat.1).range, int_index(m), m - abs(z))
                map[Nat, Int]((Nat.2 * m + Nat.1).range, int_index(m)).contains(int_index(m, m - abs(z)))
                int_range(m).contains(int_index(m, m - abs(z)))
                int_range(m).contains(z)
            }
            int_range(m).contains(z)
        }
        int_range(m).contains(z)
        fs_from_list_contains_eq(int_range(m), z)
        fs_from_list(int_range(m)).contains(z) = int_range(m).contains(z)
        fs_from_list(int_range(m)).contains(z)
        signed_range(m) = fs_from_list(int_range(m))
        signed_range(m).contains(z)
    }
}

/// An integer z lies in the signed range exactly when |z| ≤ m.
theorem signed_range_contains_abs(m: Nat, z: Int) {
    signed_range(m).contains(z) = (abs(z) <= m)
} by {
    signed_range_contains_imp_abs(m, z)
    signed_range(m).contains(z) implies abs(z) <= m
    signed_range_abs_imp_contains(m, z)
    abs(z) <= m implies signed_range(m).contains(z)
    signed_range(m).contains(z) = (abs(z) <= m)
}

/// A pair (x, y) of integers lies in the signed rectangle exactly when |x| ≤ m and |y| ≤ n.
theorem signed_rectangle_contains_abs(m: Nat, n: Nat, x: Int, y: Int) {
    signed_rectangle(m, n).contains(Pair.new(x, y)) = (abs(x) <= m and abs(y) <= n)
} by {
    signed_rectangle(m, n) = finite_set_product(signed_range(m), signed_range(n))
    pair_new_first(x, y)
    Pair.new(x, y).first = x
    pair_new_second(x, y)
    Pair.new(x, y).second = y
    finite_set_product_contains_eq(signed_range(m), signed_range(n), Pair.new(x, y))
    finite_set_product(signed_range(m), signed_range(n)).contains(Pair.new(x, y)) =
        (signed_range(m).contains(x) and signed_range(n).contains(y))
    signed_range_contains_abs(m, x)
    signed_range(m).contains(x) = (abs(x) <= m)
    signed_range_contains_abs(n, y)
    signed_range(n).contains(y) = (abs(y) <= n)
    signed_rectangle(m, n).contains(Pair.new(x, y)) = (abs(x) <= m and abs(y) <= n)
}
