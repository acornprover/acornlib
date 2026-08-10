from nat import Nat
from nat import divides_self, lt_trans, lt_not_ref, lte_trans,
    sum_lte, mul_suc_right, mul_zero_right, lte_add_left, add_comm
from list import List, sum, remove_one_cons_eq, remove_one_cons_neq,
    remove_one_contains_other
from number_theory.divisor_sum import divisor_list, nat_tau, nat_sigma,
    divisor_list_contains_implies, divisor_list_contains_of,
    divisor_list_zero, one_divides_nat
from number_theory.tau_multiplicative import positive_divisor_lte_self,
    nat_tau_le_nat_add_one, nat_tau_le_two_mul
from number_theory.sigma_multiplicative import nat_sigma_ge_self_plus_one
from number_theory.dirichlet import cofactor_image_list
from number_theory.divisor_identities import nat_sigma_eq_cofactor_image_sum
numerals Nat

// ---------------------------------------------------------------------------
// Bounds on the divisor functions.
//
// The divisor functions `tau(n)` (the divisor count) and `sigma(n)` (the
// divisor sum) are defined in `divisor_sum.ac`, with their multiplicative
// structure in `tau_multiplicative.ac` and `sigma_multiplicative.ac`.  This
// file collects the classical pointwise bounds:
//
//   (a) tau(n) >= 2 for n > 1;
//   (b) sigma(n) >= n + 1 for n > 1;
//   (c) tau(n) <= 2 sqrt(n), in the Nat-friendly forms tau(n) <= n + 1 and
//       tau(n) <= 2 n;
//   (d) sigma(n) <= n * tau(n);
//   (e) the divisor-sum reciprocity, in the Nat form
//       sum_{d | n} n / d = sigma(n).
// ---------------------------------------------------------------------------

// ---------------------------------------------------------------------------
// (a) The divisor count is at least two.
// ---------------------------------------------------------------------------

/// Removing a present element from a list reduces its length by one:
/// `list.remove_one(item).length + 1 = list.length`.
theorem list_remove_one_length[T](list: List[T], item: T) {
    list.contains(item) implies list.remove_one(item).length + Nat.1 = list.length
} by {
    define p(xs: List[T]) -> Bool {
        xs.contains(item) implies xs.remove_one(item).length + Nat.1 = xs.length
    }
    not List.nil[T].contains(item)
    p(List.nil[T])
    forall(head: T, tail: List[T]) {
        if p(tail) {
            if List.cons(head, tail).contains(item) {
                if head = item {
                    remove_one_cons_eq[T](head, tail)
                    List.cons(head, tail).remove_one(item) = tail
                    List.cons(head, tail).remove_one(item).length = tail.length
                    List.cons(head, tail).length = tail.length.suc
                    tail.length.suc = tail.length + Nat.1
                    List.cons(head, tail).remove_one(item).length + Nat.1 = List.cons(head, tail).length
                } else {
                    head != item
                    remove_one_cons_neq[T](head, tail, item)
                    List.cons(head, tail).remove_one(item) = List.cons(head, tail.remove_one(item))
                    tail.contains(item)
                    p(tail) = (tail.contains(item) implies tail.remove_one(item).length + Nat.1 = tail.length)
                    tail.contains(item) implies tail.remove_one(item).length + Nat.1 = tail.length
                    tail.remove_one(item).length + Nat.1 = tail.length
                    List.cons(head, tail.remove_one(item)).length = tail.remove_one(item).length.suc
                    List.cons(head, tail.remove_one(item)).length = tail.remove_one(item).length + Nat.1
                    List.cons(head, tail).remove_one(item).length = tail.remove_one(item).length + Nat.1
                    (tail.remove_one(item).length + Nat.1) + Nat.1 = tail.length + Nat.1
                    List.cons(head, tail).remove_one(item).length + Nat.1 = tail.length + Nat.1
                    List.cons(head, tail).length = tail.length.suc
                    tail.length.suc = tail.length + Nat.1
                    List.cons(head, tail).remove_one(item).length + Nat.1 = List.cons(head, tail).length
                }
                p(List.cons(head, tail))
            }
            p(List.cons(head, tail))
        }
    }
    p(List.nil[T]) and forall(head: T, tail: List[T]) {
        if p(tail) {
            p(List.cons(head, tail))
        }
    }
    List.induction(function(xs: List[T]) { p(xs) })
    forall(xs: List[T]) {
        p(xs)
    }
}

/// A list containing an element has length at least one.
theorem list_contains_length_ge_one[T](list: List[T], item: T) {
    list.contains(item) implies Nat.1 <= list.length
} by {
    define p(xs: List[T]) -> Bool {
        xs.contains(item) implies Nat.1 <= xs.length
    }
    not List.nil[T].contains(item)
    p(List.nil[T])
    forall(head: T, tail: List[T]) {
        if p(tail) {
            if List.cons(head, tail).contains(item) {
                if head = item {
                    List.cons(head, tail).length = tail.length.suc
                    tail.length.suc = tail.length + Nat.1
                    lte_add_left(Nat.1, Nat.0, tail.length)
                    Nat.1 + Nat.0 <= Nat.1 + tail.length
                    Nat.1 + Nat.0 = Nat.1
                    Nat.1 <= Nat.1 + tail.length
                    Nat.1 <= tail.length.suc
                    Nat.1 <= List.cons(head, tail).length
                } else {
                    tail.contains(item)
                    p(tail) = (tail.contains(item) implies Nat.1 <= tail.length)
                    tail.contains(item) implies Nat.1 <= tail.length
                    Nat.1 <= tail.length
                    List.cons(head, tail).length = tail.length.suc
                    tail.length <= tail.length.suc
                    lte_trans(Nat.1, tail.length, tail.length.suc)
                    Nat.1 <= tail.length.suc
                    Nat.1 <= List.cons(head, tail).length
                }
                p(List.cons(head, tail))
            }
            p(List.cons(head, tail))
        }
    }
    p(List.nil[T]) and forall(head: T, tail: List[T]) {
        if p(tail) {
            p(List.cons(head, tail))
        }
    }
    List.induction(function(xs: List[T]) { p(xs) })
    forall(xs: List[T]) {
        p(xs)
    }
}

/// `tau(n) >= 2` for every `n > 1`: the divisors `1` and `n` are distinct
/// members of the divisor list, so removing the divisor `1` leaves a nonempty
/// list (it still contains `n`), and the divisor list has length at least two.
theorem nat_tau_ge_two(n: Nat) {
    Nat.1 < n implies Nat.2 <= nat_tau(n)
} by {
    if Nat.1 < n {
        lt_trans(Nat.0, Nat.1, n)
        Nat.0 < n
        one_divides_nat(n)
        Nat.1.divides(n)
        divisor_list_contains_of(n, Nat.1)
        divisor_list(n).contains(Nat.1)
        divides_self(n)
        n.divides(n)
        divisor_list_contains_of(n, n)
        divisor_list(n).contains(n)
        list_remove_one_length[Nat](divisor_list(n), Nat.1)
        divisor_list(n).remove_one(Nat.1).length + Nat.1 = divisor_list(n).length
        nat_tau(n) = divisor_list(n).length
        divisor_list(n).remove_one(Nat.1).length + Nat.1 = nat_tau(n)
        if n = Nat.1 {
            Nat.1 < n
            Nat.1 < Nat.1
            lt_not_ref(Nat.1)
            false
        }
        n != Nat.1
        Nat.1 != n
        remove_one_contains_other[Nat](divisor_list(n), Nat.1, n)
        divisor_list(n).contains(n) = divisor_list(n).remove_one(Nat.1).contains(n)
        divisor_list(n).remove_one(Nat.1).contains(n)
        list_contains_length_ge_one[Nat](divisor_list(n).remove_one(Nat.1), n)
        Nat.1 <= divisor_list(n).remove_one(Nat.1).length
        lte_add_left(Nat.1, Nat.1, divisor_list(n).remove_one(Nat.1).length)
        Nat.1 + Nat.1 <= Nat.1 + divisor_list(n).remove_one(Nat.1).length
        Nat.1 + Nat.1 = Nat.2
        add_comm(Nat.1, divisor_list(n).remove_one(Nat.1).length)
        Nat.1 + divisor_list(n).remove_one(Nat.1).length = divisor_list(n).remove_one(Nat.1).length + Nat.1
        divisor_list(n).remove_one(Nat.1).length + Nat.1 = nat_tau(n)
        Nat.1 + divisor_list(n).remove_one(Nat.1).length = nat_tau(n)
        Nat.2 <= nat_tau(n)
    }
}

// ---------------------------------------------------------------------------
// (b) The divisor sum is at least n + 1.
// ---------------------------------------------------------------------------

/// `sigma(n) >= n + 1` for every `n > 1`: the divisors `n` and `1` contribute
/// `n + 1` to the divisor sum.  (Proved in `sigma_multiplicative.ac` by
/// splitting off the divisor `n` and using the proper divisor `1`.)
theorem divisor_sigma_ge_self_plus_one(n: Nat) {
    Nat.1 < n implies n + Nat.1 <= nat_sigma(n)
} by {
    if Nat.1 < n {
        nat_sigma_ge_self_plus_one(n)
        n + Nat.1 <= nat_sigma(n)
    }
}

// ---------------------------------------------------------------------------
// (c) The divisor count is at most 2 sqrt(n).
// ---------------------------------------------------------------------------

/// `tau(n) <= n + 1` for every `n`: the divisor list is unique and every entry
/// is a positive divisor of `n`, hence lies in `{1, ..., n}`.
///
/// (Proved in `tau_multiplicative.ac`.)
theorem divisor_tau_le_nat_add_one(n: Nat) {
    nat_tau(n) <= n + Nat.1
} by {
    nat_tau_le_nat_add_one(n)
    nat_tau(n) <= n + Nat.1
}

/// A Nat-friendly form of the classical bound `tau(n) <= 2 sqrt(n)`:
/// `tau(n) <= 2 n`.
///
/// The sharp bound pairs each divisor `d` with the cofactor `n / d` and counts
/// the divisors at most `sqrt(n)`, which in the naturals needs an integer
/// square root plus the filter/counting machinery of the pairing argument
/// (not developed in the library); the weakened form `tau(n) <= 2 n` is used
/// here instead (since `2 sqrt(n) <= n + 1 <= 2 n` for `n >= 1`).  (Proved in
/// `tau_multiplicative.ac`.)
theorem divisor_tau_le_two_mul(n: Nat) {
    nat_tau(n) <= Nat.2 * n
} by {
    nat_tau_le_two_mul(n)
    nat_tau(n) <= Nat.2 * n
}

// ---------------------------------------------------------------------------
// (d) The divisor sum is at most n times the divisor count.
// ---------------------------------------------------------------------------

/// A list of natural numbers all at most `n` has sum at most `n` times its
/// length.
theorem nat_sum_le_length_mul(l: List[Nat], n: Nat) {
    forall(x: Nat) { l.contains(x) implies x <= n } implies sum(l) <= n * l.length
} by {
    define p(xs: List[Nat]) -> Bool {
        forall(x: Nat) { xs.contains(x) implies x <= n } implies
            sum(xs) <= n * xs.length
    }
    forall(x: Nat) {
        if List.nil[Nat].contains(x) {
            false
        }
    }
    sum(List.nil[Nat]) = Nat.0
    List.nil[Nat].length = Nat.0
    mul_zero_right(n)
    n * Nat.0 = Nat.0
    Nat.0 <= Nat.0
    p(List.nil[Nat])
    forall(head: Nat, tail: List[Nat]) {
        if p(tail) {
            if forall(x: Nat) {
                List.cons(head, tail).contains(x) implies x <= n
            } {
                List.cons(head, tail).contains(head)
                head <= n
                forall(x: Nat) {
                    if tail.contains(x) {
                        List.cons(head, tail).contains(x)
                        x <= n
                    }
                }
                p(tail) = (forall(x: Nat) { tail.contains(x) implies x <= n } implies sum(tail) <= n * tail.length)
                (forall(x: Nat) { tail.contains(x) implies x <= n }) implies sum(tail) <= n * tail.length
                sum(tail) <= n * tail.length
                sum_lte(head, sum(tail), n, n * tail.length)
                head + sum(tail) <= n + n * tail.length
                sum(List.cons(head, tail)) = head + sum(tail)
                sum(List.cons(head, tail)) <= n + n * tail.length
                List.cons(head, tail).length = tail.length.suc
                mul_suc_right(n, tail.length)
                n * tail.length.suc = n + n * tail.length
                n + n * tail.length = n * tail.length.suc
                n * tail.length.suc = n * List.cons(head, tail).length
                n + n * tail.length = n * List.cons(head, tail).length
                lte_trans(sum(List.cons(head, tail)), n + n * tail.length,
                    n * List.cons(head, tail).length)
                sum(List.cons(head, tail)) <= n * List.cons(head, tail).length
                p(List.cons(head, tail))
            }
            p(List.cons(head, tail))
        }
    }
    p(List.nil[Nat]) and forall(head: Nat, tail: List[Nat]) {
        if p(tail) {
            p(List.cons(head, tail))
        }
    }
    List.induction(function(xs: List[Nat]) { p(xs) })
    forall(xs: List[Nat]) {
        p(xs)
    }
}

/// `sigma(n) <= n * tau(n)` for every `n`: each of the `tau(n)` positive
/// divisors of `n` is at most `n`, so the divisor sum is at most `n` times
/// the number of divisors.
theorem nat_sigma_le_nat_mul_tau(n: Nat) {
    nat_sigma(n) <= n * nat_tau(n)
} by {
    forall(d: Nat) {
        if divisor_list(n).contains(d) {
            if n = Nat.0 {
                divisor_list_zero
                divisor_list(Nat.0) = List.nil[Nat]
                not List.nil[Nat].contains(d)
                false
            }
            n != Nat.0
            divisor_list_contains_implies(n, d)
            Nat.0 < d
            d.divides(n)
            positive_divisor_lte_self(n, d)
            d <= n
        }
    }
    nat_sum_le_length_mul(divisor_list(n), n)
    sum(divisor_list(n)) <= n * divisor_list(n).length
    nat_sigma(n) = sum(divisor_list(n))
    nat_sigma(n) <= n * divisor_list(n).length
    nat_tau(n) = divisor_list(n).length
    n * nat_tau(n) = n * divisor_list(n).length
    lte_trans(nat_sigma(n), n * divisor_list(n).length, n * nat_tau(n))
    nat_sigma(n) <= n * nat_tau(n)
}

// ---------------------------------------------------------------------------
// (e) The divisor-sum reciprocity.
// ---------------------------------------------------------------------------

/// The divisor-sum reciprocity in the form the library supports:
/// `sum_{d | n} n / d = sigma(n)` for positive `n`, where the cofactor
/// `n / d` is `nat_divisor_quotient_fn(n)(d)` and `cofactor_image_list(n)`
/// is the list of cofactors.
///
/// The classical rational statement `sum_{d | n} 1 / d = sigma(n) / n` is
/// obtained by dividing both sides by `n`; the `Rat` module is not connected
/// to `divisor_list`, so the integer form above is the version the API
/// supports.  (Proved in `divisor_identities.ac`.)
theorem divisor_sigma_eq_cofactor_sum(n: Nat) {
    Nat.0 < n implies nat_sigma(n) = sum(cofactor_image_list(n))
} by {
    if Nat.0 < n {
        nat_sigma_eq_cofactor_image_sum(n)
        nat_sigma(n) = sum(cofactor_image_list(n))
    }
}
