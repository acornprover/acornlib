/// General Möbius inversion for integer-valued arithmetic functions.
///
/// The module records the Dirichlet-convolution form of the fundamental
/// identity `sum_{d | n} mu(d) = delta(n)` (both orderings of the factors),
/// the unit laws of Dirichlet convolution, and the inversion theorem itself:
/// if `g(n) = sum_{d | n} f(d)` then `f(n) = sum_{d | n} mu(d) * g(n / d)`.
/// The main proof interchanges nested divisor sums over the pair lists of
/// `dirichlet_assoc.ac` and applies the fundamental identity.
from nat import Nat, alt_induction, divides_self, mul_cancel_left, lt_trans
from int import Int
from list import List, map, sum, sum_map_of_pointwise, map_map, map_contains,
    map_contains_of_contains, sum_add, map_add, injective_map_is_unique,
    unique_same_contains_map_sum_eq
from pair import Pair, pair_eta
from data.basic.functions import is_injective_fn, compose
from number_theory.arithmetic_functions import nat_dirichlet_unit_fn
from number_theory.dirichlet import dirichlet_convolve, dirichlet_convolve_unit_right,
    dirichlet_convolve_unit_left, divisor_quotient, divisor_quotient_cofactor,
    divisor_quotient_positive, divisor_quotient_self, nat_divisor_quotient_fn,
    nat_divisor_quotient_fn_apply, cofactor_image_list, cofactor_image_list_is_unique,
    cofactor_image_list_contains_iff
from number_theory.divisor_sum import divisor_list, divisor_list_zero,
    divisor_list_contains_implies, divisor_list_contains_of, divisor_list_is_unique,
    divisors_up_to, divisors_up_to_zero, divisors_up_to_suc_yes, divisors_up_to_suc_no
from number_theory.dirichlet_assoc import right_divisor_pair_list,
    right_divisor_pair_list_contains, right_divisor_pair_list_contains_implies,
    right_divisor_pair_block, right_divisor_pair_block_from, right_divisor_pair_block_eq_from,
    right_divisor_pair_list_from, right_divisor_pair_list_unique, left_divisor_pair_list,
    left_divisor_pair_list_unique, right_left_pair_lists_same_contains,
    divisor_pair_product_divides
from number_theory.mobius_inversion import nat_mobius, nat_mobius_divisor_sum
numerals Nat
numerals Int

/// The Dirichlet unit `delta` is a right identity for Dirichlet convolution on
/// positive arguments: `(f * delta)(n) = f(n)` for `n > 0`.  The statement
/// lives in `dirichlet.ac`; it is restated here so the unit law reads as part
/// of the Möbius-inversion module.
theorem dirichlet_unit_right_identity(f: Nat -> Nat, n: Nat) {
    Nat.0 < n implies dirichlet_convolve(f, nat_dirichlet_unit_fn)(n) = f(n)
} by {
    dirichlet_convolve_unit_right(f, n)
    Nat.0 < n implies dirichlet_convolve(f, nat_dirichlet_unit_fn)(n) = f(n)
}

/// The Dirichlet unit `delta` is a left identity for Dirichlet convolution on
/// positive arguments: `(delta * f)(n) = f(n)` for `n > 0`.
theorem dirichlet_unit_left_identity(f: Nat -> Nat, n: Nat) {
    Nat.0 < n implies dirichlet_convolve(nat_dirichlet_unit_fn, f)(n) = f(n)
} by {
    dirichlet_convolve_unit_left(f, n)
    Nat.0 < n implies dirichlet_convolve(nat_dirichlet_unit_fn, f)(n) = f(n)
}

/// The constant-one arithmetic function with integer values.
define int_one_arithmetic_fn(n: Nat) -> Int {
    Int.1
}

/// The Dirichlet unit with integer values: 1 at one and 0 elsewhere.  This is
/// the integer-valued companion of `nat_dirichlet_unit_fn`.
define int_dirichlet_unit_fn(n: Nat) -> Int {
    if n = Nat.1 { Int.1 } else { Int.0 }
}

/// The integer-valued Dirichlet unit takes the value one at one.
theorem int_dirichlet_unit_fn_at_one {
    int_dirichlet_unit_fn(Nat.1) = Int.1
} by {
    int_dirichlet_unit_fn(Nat.1) = if Nat.1 = Nat.1 { Int.1 } else { Int.0 }
    (if Nat.1 = Nat.1 { Int.1 } else { Int.0 }) = Int.1
}

/// The integer-valued Dirichlet unit vanishes away from one.
theorem int_dirichlet_unit_fn_off_one(n: Nat) {
    n != Nat.1 implies int_dirichlet_unit_fn(n) = Int.0
} by {
    if n != Nat.1 {
        int_dirichlet_unit_fn(n) = if n = Nat.1 { Int.1 } else { Int.0 }
        (if n = Nat.1 { Int.1 } else { Int.0 }) = Int.0
        int_dirichlet_unit_fn(n) = Int.0
    }
}

/// The integer-valued Dirichlet unit agrees with the fundamental identity of
/// the Möbius function: `delta(n) = sum_{d | n} mu(d)`.
theorem int_dirichlet_unit_fn_eq_mobius_divisor_sum(n: Nat) {
    int_dirichlet_unit_fn(n) = sum(map(divisor_list(n), nat_mobius))
} by {
    nat_mobius_divisor_sum(n)
    sum(map(divisor_list(n), nat_mobius)) = if n = Nat.1 { Int.1 } else { Int.0 }
    int_dirichlet_unit_fn(n) = if n = Nat.1 { Int.1 } else { Int.0 }
    (if n = Nat.1 { Int.1 } else { Int.0 }) = sum(map(divisor_list(n), nat_mobius))
    int_dirichlet_unit_fn(n) = sum(map(divisor_list(n), nat_mobius))
}

/// The sum of the Möbius function over the cofactor image equals the sum over
/// the divisor list, since the cofactor map permutes the divisors of positive
/// `n`.
theorem cofactor_image_mobius_sum_eq_divisor_sum(n: Nat) {
    Nat.0 < n implies
        sum(map(cofactor_image_list(n), nat_mobius)) =
        sum(map(divisor_list(n), nat_mobius))
} by {
    if Nat.0 < n {
        cofactor_image_list_is_unique(n)
        cofactor_image_list(n).is_unique
        divisor_list_is_unique(n)
        divisor_list(n).is_unique
        forall(x: Nat) {
            cofactor_image_list_contains_iff(n, x)
            cofactor_image_list(n).contains(x) = divisor_list(n).contains(x)
        }
        unique_same_contains_map_sum_eq(cofactor_image_list(n), divisor_list(n), nat_mobius)
        sum(map(cofactor_image_list(n), nat_mobius)) =
            sum(map(divisor_list(n), nat_mobius))
    }
}

/// The Möbius function composed with the cofactor map is the function
/// `d -> mu(n / d)`.
theorem mobius_cofactor_compose(n: Nat) {
    compose(nat_mobius, nat_divisor_quotient_fn(n)) =
        function(d: Nat) { nat_mobius(divisor_quotient(n, d)) }
} by {
    forall(d: Nat) {
        compose(nat_mobius, nat_divisor_quotient_fn(n), d) =
            nat_mobius(nat_divisor_quotient_fn(n)(d))
        nat_divisor_quotient_fn_apply(n, d)
        nat_divisor_quotient_fn(n)(d) = divisor_quotient(n, d)
        compose(nat_mobius, nat_divisor_quotient_fn(n), d) =
            nat_mobius(divisor_quotient(n, d))
    }
}

/// `mu * 1 = delta`: convolving the Möbius function on the left with the
/// constant-one function gives the Dirichlet unit.  This is the fundamental
/// identity `sum_{d | n} mu(d) = delta(n)` in convolution language.
theorem mobius_convolve_one(n: Nat) {
    sum(map(divisor_list(n), function(d: Nat) {
        nat_mobius(d) * int_one_arithmetic_fn(divisor_quotient(n, d))
    })) = int_dirichlet_unit_fn(n)
} by {
    forall(d: Nat) {
        int_one_arithmetic_fn(divisor_quotient(n, d)) = Int.1
        nat_mobius(d) * int_one_arithmetic_fn(divisor_quotient(n, d)) =
            nat_mobius(d) * Int.1
        nat_mobius(d) * Int.1 = nat_mobius(d)
        nat_mobius(d) * int_one_arithmetic_fn(divisor_quotient(n, d)) = nat_mobius(d)
    }
    sum_map_of_pointwise(divisor_list(n),
        function(d: Nat) {
            nat_mobius(d) * int_one_arithmetic_fn(divisor_quotient(n, d))
        },
        nat_mobius)
    sum(map(divisor_list(n), function(d: Nat) {
        nat_mobius(d) * int_one_arithmetic_fn(divisor_quotient(n, d))
    })) = sum(map(divisor_list(n), nat_mobius))
    nat_mobius_divisor_sum(n)
    sum(map(divisor_list(n), nat_mobius)) = if n = Nat.1 { Int.1 } else { Int.0 }
    sum(map(divisor_list(n), function(d: Nat) {
        nat_mobius(d) * int_one_arithmetic_fn(divisor_quotient(n, d))
    })) = if n = Nat.1 { Int.1 } else { Int.0 }
    int_dirichlet_unit_fn(n) = if n = Nat.1 { Int.1 } else { Int.0 }
    sum(map(divisor_list(n), function(d: Nat) {
        nat_mobius(d) * int_one_arithmetic_fn(divisor_quotient(n, d))
    })) = int_dirichlet_unit_fn(n)
}

/// `1 * mu = delta`: convolving the constant-one function with the Möbius
/// function gives the Dirichlet unit.  The cofactor map `d -> n / d` permutes
/// the divisors, so this is again the fundamental identity.
theorem one_convolve_mobius(n: Nat) {
    sum(map(divisor_list(n), function(d: Nat) {
        int_one_arithmetic_fn(d) * nat_mobius(divisor_quotient(n, d))
    })) = int_dirichlet_unit_fn(n)
} by {
    forall(d: Nat) {
        int_one_arithmetic_fn(d) = Int.1
        int_one_arithmetic_fn(d) * nat_mobius(divisor_quotient(n, d)) =
            Int.1 * nat_mobius(divisor_quotient(n, d))
        Int.1 * nat_mobius(divisor_quotient(n, d)) = nat_mobius(divisor_quotient(n, d))
        int_one_arithmetic_fn(d) * nat_mobius(divisor_quotient(n, d)) =
            nat_mobius(divisor_quotient(n, d))
    }
    sum_map_of_pointwise(divisor_list(n),
        function(d: Nat) {
            int_one_arithmetic_fn(d) * nat_mobius(divisor_quotient(n, d))
        },
        function(d: Nat) { nat_mobius(divisor_quotient(n, d)) })
    sum(map(divisor_list(n), function(d: Nat) {
        int_one_arithmetic_fn(d) * nat_mobius(divisor_quotient(n, d))
    })) = sum(map(divisor_list(n),
        function(d: Nat) { nat_mobius(divisor_quotient(n, d)) }))
    map_map(divisor_list(n), nat_divisor_quotient_fn(n), nat_mobius)
    map(map(divisor_list(n), nat_divisor_quotient_fn(n)), nat_mobius) =
        map(divisor_list(n), compose(nat_mobius, nat_divisor_quotient_fn(n)))
    cofactor_image_list(n) = map(divisor_list(n), nat_divisor_quotient_fn(n))
    sum(map(cofactor_image_list(n), nat_mobius)) =
        sum(map(divisor_list(n), compose(nat_mobius, nat_divisor_quotient_fn(n))))
    forall(d: Nat) {
        if divisor_list(n).contains(d) {
            compose(nat_mobius, nat_divisor_quotient_fn(n), d) =
                nat_mobius(nat_divisor_quotient_fn(n)(d))
            nat_divisor_quotient_fn_apply(n, d)
            nat_divisor_quotient_fn(n)(d) = divisor_quotient(n, d)
            compose(nat_mobius, nat_divisor_quotient_fn(n), d) =
                nat_mobius(divisor_quotient(n, d))
        }
        divisor_list(n).contains(d) implies compose(nat_mobius, nat_divisor_quotient_fn(n), d) =
            nat_mobius(divisor_quotient(n, d))
    }
    sum_map_of_pointwise(divisor_list(n),
        compose(nat_mobius, nat_divisor_quotient_fn(n)),
        function(d: Nat) { nat_mobius(divisor_quotient(n, d)) })
    sum(map(divisor_list(n), compose(nat_mobius, nat_divisor_quotient_fn(n)))) =
        sum(map(divisor_list(n), function(d: Nat) { nat_mobius(divisor_quotient(n, d)) }))
    sum(map(cofactor_image_list(n), nat_mobius)) =
        sum(map(divisor_list(n), function(d: Nat) { nat_mobius(divisor_quotient(n, d)) }))
    if n = Nat.0 {
        divisor_list_zero
        divisor_list(n) = List.nil[Nat]
        cofactor_image_list(n) = map(divisor_list(n), nat_divisor_quotient_fn(n))
        map(List.nil[Nat], nat_divisor_quotient_fn(n)) = List.nil[Nat]
        cofactor_image_list(n) = List.nil[Nat]
        sum(map(List.nil[Nat], nat_mobius)) = Int.0
        sum(map(cofactor_image_list(n), nat_mobius)) = Int.0
        sum(map(divisor_list(n), function(d: Nat) {
            int_one_arithmetic_fn(d) * nat_mobius(divisor_quotient(n, d))
        })) = Int.0
        int_dirichlet_unit_fn(n) = if n = Nat.1 { Int.1 } else { Int.0 }
        int_dirichlet_unit_fn(n) = Int.0
        sum(map(divisor_list(n), function(d: Nat) {
            int_one_arithmetic_fn(d) * nat_mobius(divisor_quotient(n, d))
        })) = int_dirichlet_unit_fn(n)
    } else {
        n != Nat.0
        Nat.0 < n
        cofactor_image_mobius_sum_eq_divisor_sum(n)
        sum(map(cofactor_image_list(n), nat_mobius)) =
            sum(map(divisor_list(n), nat_mobius))
        sum(map(divisor_list(n), function(d: Nat) {
            int_one_arithmetic_fn(d) * nat_mobius(divisor_quotient(n, d))
        })) = sum(map(divisor_list(n), nat_mobius))
        nat_mobius_divisor_sum(n)
        sum(map(divisor_list(n), nat_mobius)) = if n = Nat.1 { Int.1 } else { Int.0 }
        sum(map(divisor_list(n), function(d: Nat) {
            int_one_arithmetic_fn(d) * nat_mobius(divisor_quotient(n, d))
        })) = if n = Nat.1 { Int.1 } else { Int.0 }
        int_dirichlet_unit_fn(n) = if n = Nat.1 { Int.1 } else { Int.0 }
        sum(map(divisor_list(n), function(d: Nat) {
            int_one_arithmetic_fn(d) * nat_mobius(divisor_quotient(n, d))
        })) = int_dirichlet_unit_fn(n)
    }
}

/// Swap the two coordinates of a pair of natural numbers.
define pair_swap(p: Pair[Nat, Nat]) -> Pair[Nat, Nat] {
    Pair.new(p.second, p.first)
}

/// Swapping a freshly built pair exchanges the coordinates.
theorem pair_swap_new(x: Nat, y: Nat) {
    pair_swap(Pair.new(x, y)) = Pair.new(y, x)
} by {
    Pair.new(x, y).first = x
    Pair.new(x, y).second = y
    pair_swap(Pair.new(x, y)) = Pair.new(Pair.new(x, y).second, Pair.new(x, y).first)
    pair_swap(Pair.new(x, y)) = Pair.new(y, x)
}

/// Swapping is an involution.
theorem pair_swap_involution(p: Pair[Nat, Nat]) {
    pair_swap(pair_swap(p)) = p
} by {
    pair_eta(p)
    Pair.new(p.first, p.second) = p
    pair_swap(p) = pair_swap(Pair.new(p.first, p.second))
    pair_swap_new(p.first, p.second)
    pair_swap(Pair.new(p.first, p.second)) = Pair.new(p.second, p.first)
    pair_swap(p) = Pair.new(p.second, p.first)
    pair_swap(pair_swap(p)) = pair_swap(Pair.new(p.second, p.first))
    pair_swap_new(p.second, p.first)
    pair_swap(Pair.new(p.second, p.first)) = Pair.new(p.first, p.second)
    pair_swap(pair_swap(p)) = Pair.new(p.first, p.second)
    pair_swap(pair_swap(p)) = p
}

/// Swapping is injective.
theorem pair_swap_injective {
    is_injective_fn(pair_swap)
} by {
    is_injective_fn(pair_swap) = forall(x: Pair[Nat, Nat], y: Pair[Nat, Nat]) {
        pair_swap(x) = pair_swap(y) implies x = y
    }
    forall(p: Pair[Nat, Nat], q: Pair[Nat, Nat]) {
        if pair_swap(p) = pair_swap(q) {
            pair_swap(pair_swap(p)) = pair_swap(pair_swap(q))
            pair_swap_involution(p)
            pair_swap_involution(q)
            pair_swap(pair_swap(p)) = p
            pair_swap(pair_swap(q)) = q
            p = q
        }
        pair_swap(p) = pair_swap(q) implies p = q
    }
    is_injective_fn(pair_swap)
}

/// A listed pair in the right divisor-pair list of a positive `n` has its
/// swapped pair listed as well, since `d * e | n` is symmetric.
theorem right_pair_list_swap_forward(n: Nat, p: Pair[Nat, Nat]) {
    Nat.0 < n and right_divisor_pair_list(n).contains(p) implies
        right_divisor_pair_list(n).contains(pair_swap(p))
} by {
    if Nat.0 < n and right_divisor_pair_list(n).contains(p) {
        right_divisor_pair_list_contains_implies(n, p)
        divisor_list(n).contains(p.first)
        divisor_list(divisor_quotient(n, p.first)).contains(p.second)
        divisor_list_contains_implies(n, p.first)
        Nat.0 < p.first and p.first.divides(n)
        Nat.0 < p.first
        p.first.divides(n)
        divisor_list_contains_implies(divisor_quotient(n, p.first), p.second)
        Nat.0 < p.second and p.second.divides(divisor_quotient(n, p.first))
        Nat.0 < p.second
        p.second.divides(divisor_quotient(n, p.first))
        divisor_pair_product_divides(n, p.first, p.second)
        (p.first * p.second).divides(n)
        let c: Nat satisfy { (p.first * p.second) * c = n }
        (p.first * p.second) * c = n
        p.second * (p.first * c) = (p.first * p.second) * c
        p.second * (p.first * c) = n
        p.second.divides(n) = exists(c0: Nat) { p.second * c0 = n }
        p.second.divides(n)
        divisor_quotient_cofactor(n, p.second)
        p.second * divisor_quotient(n, p.second) = n
        p.second * divisor_quotient(n, p.second) = p.second * (p.first * c)
        p.second != Nat.0
        mul_cancel_left(p.second, divisor_quotient(n, p.second), p.first * c)
        divisor_quotient(n, p.second) = p.first * c
        p.first.divides(divisor_quotient(n, p.second)) =
            exists(c1: Nat) { p.first * c1 = divisor_quotient(n, p.second) }
        p.first.divides(divisor_quotient(n, p.second))
        divisor_quotient_positive(n, p.second)
        Nat.0 < divisor_quotient(n, p.second)
        divisor_list_contains_of(divisor_quotient(n, p.second), p.first)
        divisor_list(divisor_quotient(n, p.second)).contains(p.first)
        divisor_list_contains_of(n, p.second)
        divisor_list(n).contains(p.second)
        right_divisor_pair_list_contains(n, p.second, p.first)
        right_divisor_pair_list(n).contains(Pair.new(p.second, p.first))
        pair_swap_new(p.first, p.second)
        pair_swap(Pair.new(p.first, p.second)) = Pair.new(p.second, p.first)
        pair_eta(p)
        Pair.new(p.first, p.second) = p
        pair_swap(p) = pair_swap(Pair.new(p.first, p.second))
        right_divisor_pair_list(n).contains(pair_swap(p))
    }
}

/// The right divisor-pair list of a positive `n` is closed under swapping the
/// pair coordinates: `(d, e)` is a listed pair exactly when `(e, d)` is.
theorem right_pair_list_swap_contains(n: Nat, p: Pair[Nat, Nat]) {
    Nat.0 < n implies
        right_divisor_pair_list(n).contains(p) =
        right_divisor_pair_list(n).contains(pair_swap(p))
} by {
    if Nat.0 < n {
        if right_divisor_pair_list(n).contains(p) {
            right_pair_list_swap_forward(n, p)
            right_divisor_pair_list(n).contains(pair_swap(p))
        }
        if right_divisor_pair_list(n).contains(pair_swap(p)) {
            right_pair_list_swap_forward(n, pair_swap(p))
            Nat.0 < n and right_divisor_pair_list(n).contains(pair_swap(p)) implies right_divisor_pair_list(n).contains(pair_swap(pair_swap(p)))
            right_divisor_pair_list(n).contains(pair_swap(pair_swap(p)))
            pair_swap_involution(p)
            pair_swap(pair_swap(p)) = p
            right_divisor_pair_list(n).contains(p)
        }
        right_divisor_pair_list(n).contains(p) =
            right_divisor_pair_list(n).contains(pair_swap(p))
    }
}

/// The pair term of a right-nested expansion: the value `a(p.first) * b(p.second)`
/// attached to the pair `p`.
define int_pair_term(a: Nat -> Int, b: Nat -> Int) -> (Pair[Nat, Nat] -> Int) {
    function(p: Pair[Nat, Nat]) { a(p.first) * b(p.second) }
}

/// A scaled inner divisor sum at a fixed outer divisor `d | n` expands to the
/// sum over the fixed-left right pair block for `d`.
theorem int_right_block_expand(a: Nat -> Int, b: Nat -> Int, n: Nat, d: Nat) {
    Nat.0 < n and d.divides(n) implies
        a(d) * sum(map(divisor_list(divisor_quotient(n, d)), b)) =
        sum(map(right_divisor_pair_block(n, d), int_pair_term(a, b)))
} by {
    let q: Nat = divisor_quotient(n, d)
    define pred(bound: Nat) -> Bool {
        Nat.0 < n and d.divides(n) implies
            a(d) * sum(map(divisors_up_to(q, bound), b)) =
            sum(map(right_divisor_pair_block_from(n, d, divisors_up_to(q, bound)),
                int_pair_term(a, b)))
    }
    divisors_up_to_zero(q)
    divisors_up_to(q, Nat.0) = List.nil[Nat]
    map(List.nil[Nat], b) = List.nil[Int]
    sum(List.nil[Int]) = Int.0
    a(d) * Int.0 = Int.0
    right_divisor_pair_block_from(n, d, List.nil[Nat]) = List.nil[Pair[Nat, Nat]]
    map(List.nil[Pair[Nat, Nat]], int_pair_term(a, b)) = List.nil[Int]
    sum(List.nil[Int]) = Int.0
    pred(Nat.0)
    forall(j: Nat) {
        if pred(j) {
            pred(j) = (Nat.0 < n and d.divides(n) implies
                a(d) * sum(map(divisors_up_to(q, j), b)) =
                sum(map(right_divisor_pair_block_from(n, d, divisors_up_to(q, j)),
                    int_pair_term(a, b))))
            if Nat.0 < n and d.divides(n) {
                if j.suc.divides(q) {
                    divisors_up_to_suc_yes(q, j)
                    divisors_up_to(q, j.suc) = List.cons(j.suc, divisors_up_to(q, j))
                    map(List.cons(j.suc, divisors_up_to(q, j)), b) =
                        List.cons(b(j.suc), map(divisors_up_to(q, j), b))
                    map(divisors_up_to(q, j.suc), b) =
                        map(List.cons(j.suc, divisors_up_to(q, j)), b)
                    sum(map(divisors_up_to(q, j.suc), b)) =
                        sum(List.cons(b(j.suc), map(divisors_up_to(q, j), b)))
                    sum(List.cons(b(j.suc), map(divisors_up_to(q, j), b))) =
                        b(j.suc) + sum(map(divisors_up_to(q, j), b))
                    sum(map(divisors_up_to(q, j.suc), b)) =
                        b(j.suc) + sum(map(divisors_up_to(q, j), b))
                    a(d) * sum(map(divisors_up_to(q, j.suc), b)) =
                        a(d) * (b(j.suc) + sum(map(divisors_up_to(q, j), b)))
                    a(d) * (b(j.suc) + sum(map(divisors_up_to(q, j), b))) =
                        a(d) * b(j.suc) + a(d) * sum(map(divisors_up_to(q, j), b))
                    right_divisor_pair_block_from(n, d, List.cons(j.suc, divisors_up_to(q, j))) =
                        List.cons(Pair.new(d, j.suc),
                            right_divisor_pair_block_from(n, d, divisors_up_to(q, j)))
                    right_divisor_pair_block_from(n, d, divisors_up_to(q, j.suc)) =
                        right_divisor_pair_block_from(n, d, List.cons(j.suc, divisors_up_to(q, j)))
                    map(right_divisor_pair_block_from(n, d, divisors_up_to(q, j.suc)),
                            int_pair_term(a, b)) =
                        map(List.cons(Pair.new(d, j.suc),
                                right_divisor_pair_block_from(n, d, divisors_up_to(q, j))),
                            int_pair_term(a, b))
                    map(List.cons(Pair.new(d, j.suc),
                                right_divisor_pair_block_from(n, d, divisors_up_to(q, j))),
                            int_pair_term(a, b)) =
                        List.cons(int_pair_term(a, b)(Pair.new(d, j.suc)),
                            map(right_divisor_pair_block_from(n, d, divisors_up_to(q, j)),
                                int_pair_term(a, b)))
                    sum(map(right_divisor_pair_block_from(n, d, divisors_up_to(q, j.suc)),
                            int_pair_term(a, b))) =
                        sum(List.cons(int_pair_term(a, b)(Pair.new(d, j.suc)),
                            map(right_divisor_pair_block_from(n, d, divisors_up_to(q, j)),
                                int_pair_term(a, b))))
                    sum(List.cons(int_pair_term(a, b)(Pair.new(d, j.suc)),
                            map(right_divisor_pair_block_from(n, d, divisors_up_to(q, j)),
                                int_pair_term(a, b)))) =
                        int_pair_term(a, b)(Pair.new(d, j.suc)) +
                            sum(map(right_divisor_pair_block_from(n, d, divisors_up_to(q, j)),
                                int_pair_term(a, b)))
                    Pair.new(d, j.suc).first = d
                    Pair.new(d, j.suc).second = j.suc
                    int_pair_term(a, b)(Pair.new(d, j.suc)) = a(d) * b(j.suc)
                    pred(j)
                    a(d) * sum(map(divisors_up_to(q, j), b)) =
                        sum(map(right_divisor_pair_block_from(n, d, divisors_up_to(q, j)),
                            int_pair_term(a, b)))
                    a(d) * sum(map(divisors_up_to(q, j.suc), b)) =
                        sum(map(right_divisor_pair_block_from(n, d, divisors_up_to(q, j.suc)),
                            int_pair_term(a, b)))
                } else {
                    not j.suc.divides(q)
                    divisors_up_to_suc_no(q, j)
                    divisors_up_to(q, j.suc) = divisors_up_to(q, j)
                    pred(j)
                    a(d) * sum(map(divisors_up_to(q, j), b)) =
                        sum(map(right_divisor_pair_block_from(n, d, divisors_up_to(q, j)),
                            int_pair_term(a, b)))
                    a(d) * sum(map(divisors_up_to(q, j.suc), b)) =
                        sum(map(right_divisor_pair_block_from(n, d, divisors_up_to(q, j.suc)),
                            int_pair_term(a, b)))
                }
            }
            pred(j.suc) = (Nat.0 < n and d.divides(n) implies
                a(d) * sum(map(divisors_up_to(q, j.suc), b)) =
                sum(map(right_divisor_pair_block_from(n, d, divisors_up_to(q, j.suc)),
                    int_pair_term(a, b))))
            pred(j.suc)
        }
    }
    forall(bound: Nat) { pred(bound) implies pred(bound.suc) }
    pred(Nat.0) and forall(bound: Nat) { pred(bound) implies pred(bound.suc) }
    alt_induction(pred)
    forall(bound: Nat) { pred(bound) }
    pred(q)
    pred(q) = (Nat.0 < n and d.divides(n) implies
        a(d) * sum(map(divisors_up_to(q, q), b)) =
        sum(map(right_divisor_pair_block_from(n, d, divisors_up_to(q, q)),
            int_pair_term(a, b))))
    if Nat.0 < n and d.divides(n) {
        a(d) * sum(map(divisors_up_to(q, q), b)) =
            sum(map(right_divisor_pair_block_from(n, d, divisors_up_to(q, q)),
                int_pair_term(a, b)))
        divisor_list(q) = divisors_up_to(q, q)
        map(divisor_list(q), b) = map(divisors_up_to(q, q), b)
        a(d) * sum(map(divisor_list(q), b)) =
            sum(map(right_divisor_pair_block_from(n, d, divisors_up_to(q, q)),
                int_pair_term(a, b)))
        right_divisor_pair_block_eq_from(n, d)
        right_divisor_pair_block(n, d) =
            right_divisor_pair_block_from(n, d, divisor_list(divisor_quotient(n, d)))
        divisor_list(divisor_quotient(n, d)) = divisor_list(q)
        right_divisor_pair_block(n, d) = right_divisor_pair_block_from(n, d, divisor_list(q))
        sum(map(right_divisor_pair_block(n, d), int_pair_term(a, b))) =
            sum(map(right_divisor_pair_block_from(n, d, divisor_list(q)),
                int_pair_term(a, b)))
        sum(map(right_divisor_pair_block_from(n, d, divisors_up_to(q, q)),
                int_pair_term(a, b))) =
            sum(map(right_divisor_pair_block_from(n, d, divisor_list(q)),
                int_pair_term(a, b)))
        a(d) * sum(map(divisor_list(q), b)) =
            sum(map(right_divisor_pair_block(n, d), int_pair_term(a, b)))
        map(divisor_list(divisor_quotient(n, d)), b) = map(divisor_list(q), b)
        a(d) * sum(map(divisor_list(divisor_quotient(n, d)), b)) =
            sum(map(right_divisor_pair_block(n, d), int_pair_term(a, b)))
    }
    Nat.0 < n and d.divides(n) implies a(d) * sum(map(divisor_list(divisor_quotient(n, d)), b)) =
        sum(map(right_divisor_pair_block(n, d), int_pair_term(a, b)))
}

/// The outer divisor sum of scaled inner divisor sums expands to the sum over
/// the right divisor-pair list: `sum_{d | n} a(d) * sum_{e | n/d} b(e)` equals
/// `sum_{(d, e) in pairs} a(d) * b(e)`.
theorem int_right_expand(a: Nat -> Int, b: Nat -> Int, n: Nat) {
    Nat.0 < n implies
        sum(map(divisor_list(n), function(d: Nat) {
            a(d) * sum(map(divisor_list(divisor_quotient(n, d)), b))
        })) =
        sum(map(right_divisor_pair_list(n), int_pair_term(a, b)))
} by {
    define outer_fn(n0: Nat) -> (Nat -> Int) {
        function(d: Nat) {
            a(d) * sum(map(divisor_list(divisor_quotient(n0, d)), b))
        }
    }
    define pred(bound: Nat) -> Bool {
        Nat.0 < n implies
            sum(map(divisors_up_to(n, bound), outer_fn(n))) =
            sum(map(right_divisor_pair_list_from(n, divisors_up_to(n, bound)),
                int_pair_term(a, b)))
    }
    divisors_up_to_zero(n)
    divisors_up_to(n, Nat.0) = List.nil[Nat]
    map(List.nil[Nat], outer_fn(n)) = List.nil[Int]
    sum(List.nil[Int]) = Int.0
    right_divisor_pair_list_from(n, List.nil[Nat]) = List.nil[Pair[Nat, Nat]]
    map(List.nil[Pair[Nat, Nat]], int_pair_term(a, b)) = List.nil[Int]
    sum(List.nil[Int]) = Int.0
    pred(Nat.0)
    forall(j: Nat) {
        if pred(j) {
            pred(j) = (Nat.0 < n implies
                sum(map(divisors_up_to(n, j), outer_fn(n))) =
                sum(map(right_divisor_pair_list_from(n, divisors_up_to(n, j)),
                    int_pair_term(a, b))))
            if Nat.0 < n {
                if j.suc.divides(n) {
                    divisors_up_to_suc_yes(n, j)
                    divisors_up_to(n, j.suc) = List.cons(j.suc, divisors_up_to(n, j))
                    map(List.cons(j.suc, divisors_up_to(n, j)), outer_fn(n)) =
                        List.cons(outer_fn(n)(j.suc), map(divisors_up_to(n, j), outer_fn(n)))
                    map(divisors_up_to(n, j.suc), outer_fn(n)) =
                        map(List.cons(j.suc, divisors_up_to(n, j)), outer_fn(n))
                    sum(map(divisors_up_to(n, j.suc), outer_fn(n))) =
                        sum(List.cons(outer_fn(n)(j.suc), map(divisors_up_to(n, j), outer_fn(n))))
                    sum(List.cons(outer_fn(n)(j.suc), map(divisors_up_to(n, j), outer_fn(n)))) =
                        outer_fn(n)(j.suc) + sum(map(divisors_up_to(n, j), outer_fn(n)))
                    outer_fn(n)(j.suc) =
                        a(j.suc) * sum(map(divisor_list(divisor_quotient(n, j.suc)), b))
                    int_right_block_expand(a, b, n, j.suc)
                    a(j.suc) * sum(map(divisor_list(divisor_quotient(n, j.suc)), b)) =
                        sum(map(right_divisor_pair_block(n, j.suc), int_pair_term(a, b)))
                    outer_fn(n)(j.suc) =
                        sum(map(right_divisor_pair_block(n, j.suc), int_pair_term(a, b)))
                    pred(j)
                    sum(map(divisors_up_to(n, j), outer_fn(n))) =
                        sum(map(right_divisor_pair_list_from(n, divisors_up_to(n, j)),
                            int_pair_term(a, b)))
                    right_divisor_pair_list_from(n, List.cons(j.suc, divisors_up_to(n, j))) =
                        right_divisor_pair_block(n, j.suc) +
                            right_divisor_pair_list_from(n, divisors_up_to(n, j))
                    right_divisor_pair_list_from(n, divisors_up_to(n, j.suc)) =
                        right_divisor_pair_list_from(n, List.cons(j.suc, divisors_up_to(n, j)))
                    map(right_divisor_pair_block(n, j.suc) +
                            right_divisor_pair_list_from(n, divisors_up_to(n, j)),
                        int_pair_term(a, b)) =
                        map(right_divisor_pair_block(n, j.suc), int_pair_term(a, b)) +
                        map(right_divisor_pair_list_from(n, divisors_up_to(n, j)),
                            int_pair_term(a, b))
                    sum(map(right_divisor_pair_list_from(n, divisors_up_to(n, j.suc)),
                            int_pair_term(a, b))) =
                        sum(map(right_divisor_pair_block(n, j.suc) +
                                right_divisor_pair_list_from(n, divisors_up_to(n, j)),
                            int_pair_term(a, b)))
                    map_add(right_divisor_pair_block(n, j.suc),
                        right_divisor_pair_list_from(n, divisors_up_to(n, j)),
                        int_pair_term(a, b))
                    map(right_divisor_pair_block(n, j.suc) +
                            right_divisor_pair_list_from(n, divisors_up_to(n, j)),
                        int_pair_term(a, b)) =
                        map(right_divisor_pair_block(n, j.suc), int_pair_term(a, b)) +
                        map(right_divisor_pair_list_from(n, divisors_up_to(n, j)),
                            int_pair_term(a, b))
                    sum_add(map(right_divisor_pair_block(n, j.suc), int_pair_term(a, b)),
                        map(right_divisor_pair_list_from(n, divisors_up_to(n, j)),
                            int_pair_term(a, b)))
                    sum(map(right_divisor_pair_block(n, j.suc), int_pair_term(a, b)) +
                            map(right_divisor_pair_list_from(n, divisors_up_to(n, j)),
                                int_pair_term(a, b))) =
                        sum(map(right_divisor_pair_block(n, j.suc), int_pair_term(a, b))) +
                        sum(map(right_divisor_pair_list_from(n, divisors_up_to(n, j)),
                            int_pair_term(a, b)))
                    sum(map(right_divisor_pair_block(n, j.suc) +
                                right_divisor_pair_list_from(n, divisors_up_to(n, j)),
                            int_pair_term(a, b))) =
                        sum(map(right_divisor_pair_block(n, j.suc), int_pair_term(a, b))) +
                        sum(map(right_divisor_pair_list_from(n, divisors_up_to(n, j)),
                            int_pair_term(a, b)))
                    sum(map(divisors_up_to(n, j.suc), outer_fn(n))) =
                        sum(map(right_divisor_pair_list_from(n, divisors_up_to(n, j.suc)),
                            int_pair_term(a, b)))
                } else {
                    not j.suc.divides(n)
                    divisors_up_to_suc_no(n, j)
                    divisors_up_to(n, j.suc) = divisors_up_to(n, j)
                    pred(j)
                    sum(map(divisors_up_to(n, j), outer_fn(n))) =
                        sum(map(right_divisor_pair_list_from(n, divisors_up_to(n, j)),
                            int_pair_term(a, b)))
                    sum(map(divisors_up_to(n, j.suc), outer_fn(n))) =
                        sum(map(right_divisor_pair_list_from(n, divisors_up_to(n, j.suc)),
                            int_pair_term(a, b)))
                }
            }
            pred(j.suc) = (Nat.0 < n implies
                sum(map(divisors_up_to(n, j.suc), outer_fn(n))) =
                sum(map(right_divisor_pair_list_from(n, divisors_up_to(n, j.suc)),
                    int_pair_term(a, b))))
            pred(j.suc)
        }
    }
    forall(bound: Nat) { pred(bound) implies pred(bound.suc) }
    pred(Nat.0) and forall(bound: Nat) { pred(bound) implies pred(bound.suc) }
    alt_induction(pred)
    forall(bound: Nat) { pred(bound) }
    pred(n)
    pred(n) = (Nat.0 < n implies
        sum(map(divisors_up_to(n, n), outer_fn(n))) =
        sum(map(right_divisor_pair_list_from(n, divisors_up_to(n, n)),
            int_pair_term(a, b))))
    if Nat.0 < n {
        sum(map(divisors_up_to(n, n), outer_fn(n))) =
            sum(map(right_divisor_pair_list_from(n, divisors_up_to(n, n)),
                int_pair_term(a, b)))
        divisor_list(n) = divisors_up_to(n, n)
        sum(map(divisor_list(n), outer_fn(n))) =
            sum(map(right_divisor_pair_list_from(n, divisors_up_to(n, n)),
                int_pair_term(a, b)))
        right_divisor_pair_list(n) = right_divisor_pair_list_from(n, divisor_list(n))
        sum(map(right_divisor_pair_list(n), int_pair_term(a, b))) =
            sum(map(right_divisor_pair_list_from(n, divisor_list(n)),
                int_pair_term(a, b)))
        map(divisor_list(n), outer_fn(n)) =
            map(divisor_list(n), function(d: Nat) {
                a(d) * sum(map(divisor_list(divisor_quotient(n, d)), b))
            })
        sum(map(divisor_list(n), outer_fn(n))) =
            sum(map(divisor_list(n), function(d: Nat) {
                a(d) * sum(map(divisor_list(divisor_quotient(n, d)), b))
            }))
        sum(map(divisor_list(n), function(d: Nat) {
            a(d) * sum(map(divisor_list(divisor_quotient(n, d)), b))
        })) =
            sum(map(right_divisor_pair_list(n), int_pair_term(a, b)))
    }
    Nat.0 < n implies sum(map(divisor_list(n), function(d: Nat) {
        a(d) * sum(map(divisor_list(divisor_quotient(n, d)), b))
    })) =
        sum(map(right_divisor_pair_list(n), int_pair_term(a, b)))
}

/// The swapped pair term: `b(p.second) * a(p.first)`.
define int_pair_term_swapped(a: Nat -> Int, b: Nat -> Int) -> (Pair[Nat, Nat] -> Int) {
    function(p: Pair[Nat, Nat]) { b(p.second) * a(p.first) }
}

/// The right divisor-pair list with each pair's coordinates swapped.
define swapped_right_pair_list(n: Nat) -> List[Pair[Nat, Nat]] {
    map(right_divisor_pair_list(n), pair_swap)
}

/// The swapped right pair list has the same membership as the right pair list,
/// since the pair set is closed under swapping.
theorem swapped_right_pair_list_same_contains(n: Nat, x: Pair[Nat, Nat]) {
    Nat.0 < n implies
        swapped_right_pair_list(n).contains(x) = right_divisor_pair_list(n).contains(x)
} by {
    if Nat.0 < n {
        swapped_right_pair_list(n) = map(right_divisor_pair_list(n), pair_swap)
        if swapped_right_pair_list(n).contains(x) {
            map_contains(right_divisor_pair_list(n), pair_swap, x)
            let p: Pair[Nat, Nat] satisfy {
                right_divisor_pair_list(n).contains(p) and pair_swap(p) = x
            }
            right_pair_list_swap_forward(n, p)
            right_divisor_pair_list(n).contains(pair_swap(p))
            right_divisor_pair_list(n).contains(x)
        }
        if right_divisor_pair_list(n).contains(x) {
            right_pair_list_swap_forward(n, x)
            right_divisor_pair_list(n).contains(pair_swap(x))
            map_contains_of_contains(right_divisor_pair_list(n), pair_swap, pair_swap(x))
            map(right_divisor_pair_list(n), pair_swap).contains(pair_swap(pair_swap(x)))
            pair_swap_involution(x)
            pair_swap(pair_swap(x)) = x
            map(right_divisor_pair_list(n), pair_swap).contains(x)
            swapped_right_pair_list(n).contains(x)
        }
        swapped_right_pair_list(n).contains(x) = right_divisor_pair_list(n).contains(x)
    }
}

/// The swapped right pair list is unique, since swapping is injective.
theorem swapped_right_pair_list_unique(n: Nat) {
    swapped_right_pair_list(n).is_unique
} by {
    right_divisor_pair_list_unique(n)
    right_divisor_pair_list(n).is_unique
    pair_swap_injective
    is_injective_fn(pair_swap)
    injective_map_is_unique(right_divisor_pair_list(n), pair_swap)
    map(right_divisor_pair_list(n), pair_swap).is_unique
    swapped_right_pair_list(n) = map(right_divisor_pair_list(n), pair_swap)
    swapped_right_pair_list(n).is_unique
}

/// Reindexing the swapped pair list back recovers the unswapped pair term:
/// `sum over swapped pairs of b(second) * a(first)` equals
/// `sum over pairs of b(first) * a(second)`.
theorem swapped_list_reindex_sum(a: Nat -> Int, b: Nat -> Int, n: Nat) {
    sum(map(swapped_right_pair_list(n), int_pair_term_swapped(a, b))) =
        sum(map(right_divisor_pair_list(n), int_pair_term(b, a)))
} by {
    map_map(right_divisor_pair_list(n), pair_swap, int_pair_term_swapped(a, b))
    map(map(right_divisor_pair_list(n), pair_swap), int_pair_term_swapped(a, b)) =
        map(right_divisor_pair_list(n), compose(int_pair_term_swapped(a, b), pair_swap))
    swapped_right_pair_list(n) = map(right_divisor_pair_list(n), pair_swap)
    sum(map(swapped_right_pair_list(n), int_pair_term_swapped(a, b))) =
        sum(map(right_divisor_pair_list(n), compose(int_pair_term_swapped(a, b), pair_swap)))
    forall(x: Pair[Nat, Nat]) {
        compose(int_pair_term_swapped(a, b), pair_swap, x) =
            int_pair_term_swapped(a, b)(pair_swap(x))
        pair_swap(x).first = x.second
        pair_swap(x).second = x.first
        int_pair_term_swapped(a, b)(pair_swap(x)) = b(pair_swap(x).second) * a(pair_swap(x).first)
        int_pair_term_swapped(a, b)(pair_swap(x)) = b(x.first) * a(x.second)
        int_pair_term(b, a)(x) = b(x.first) * a(x.second)
        compose(int_pair_term_swapped(a, b), pair_swap, x) = int_pair_term(b, a)(x)
    }
    sum_map_of_pointwise(right_divisor_pair_list(n),
        compose(int_pair_term_swapped(a, b), pair_swap), int_pair_term(b, a))
    sum(map(right_divisor_pair_list(n), compose(int_pair_term_swapped(a, b), pair_swap))) =
        sum(map(right_divisor_pair_list(n), int_pair_term(b, a)))
    sum(map(swapped_right_pair_list(n), int_pair_term_swapped(a, b))) =
        sum(map(right_divisor_pair_list(n), int_pair_term(b, a)))
}

/// Interchanging nested divisor sums: `sum_{d | n} a(d) * sum_{e | n/d} b(e)`
/// equals `sum_{e | n} b(e) * sum_{d | n/e} a(d)` for positive `n`.
theorem int_divisor_sum_interchange(a: Nat -> Int, b: Nat -> Int, n: Nat) {
    Nat.0 < n implies
        sum(map(divisor_list(n), function(d: Nat) {
            a(d) * sum(map(divisor_list(divisor_quotient(n, d)), b))
        })) =
        sum(map(divisor_list(n), function(e: Nat) {
            b(e) * sum(map(divisor_list(divisor_quotient(n, e)), a))
        }))
} by {
    if Nat.0 < n {
        int_right_expand(a, b, n)
        sum(map(divisor_list(n), function(d: Nat) {
            a(d) * sum(map(divisor_list(divisor_quotient(n, d)), b))
        })) = sum(map(right_divisor_pair_list(n), int_pair_term(a, b)))
        right_divisor_pair_list_unique(n)
        right_divisor_pair_list(n).is_unique
        left_divisor_pair_list_unique(n)
        left_divisor_pair_list(n).is_unique
        forall(x: Pair[Nat, Nat]) {
            right_left_pair_lists_same_contains(n, x)
            right_divisor_pair_list(n).contains(x) = left_divisor_pair_list(n).contains(x)
            left_divisor_pair_list(n).contains(x) = right_divisor_pair_list(n).contains(x)
        }
        unique_same_contains_map_sum_eq(right_divisor_pair_list(n), left_divisor_pair_list(n),
            int_pair_term(a, b))
        sum(map(right_divisor_pair_list(n), int_pair_term(a, b))) =
            sum(map(left_divisor_pair_list(n), int_pair_term(a, b)))
        sum(map(divisor_list(n), function(d: Nat) {
            a(d) * sum(map(divisor_list(divisor_quotient(n, d)), b))
        })) = sum(map(left_divisor_pair_list(n), int_pair_term(a, b)))
        forall(x: Pair[Nat, Nat]) {
            if left_divisor_pair_list(n).contains(x) {
                int_pair_term(a, b)(x) = a(x.first) * b(x.second)
                a(x.first) * b(x.second) = b(x.second) * a(x.first)
                int_pair_term_swapped(a, b)(x) = b(x.second) * a(x.first)
                int_pair_term(a, b)(x) = int_pair_term_swapped(a, b)(x)
            }
            left_divisor_pair_list(n).contains(x) implies int_pair_term(a, b)(x) = int_pair_term_swapped(a, b)(x)
        }
        sum_map_of_pointwise(left_divisor_pair_list(n), int_pair_term(a, b),
            int_pair_term_swapped(a, b))
        sum(map(left_divisor_pair_list(n), int_pair_term(a, b))) =
            sum(map(left_divisor_pair_list(n), int_pair_term_swapped(a, b)))
        sum(map(divisor_list(n), function(d: Nat) {
            a(d) * sum(map(divisor_list(divisor_quotient(n, d)), b))
        })) = sum(map(left_divisor_pair_list(n), int_pair_term_swapped(a, b)))
        unique_same_contains_map_sum_eq(left_divisor_pair_list(n), right_divisor_pair_list(n),
            int_pair_term_swapped(a, b))
        sum(map(left_divisor_pair_list(n), int_pair_term_swapped(a, b))) =
            sum(map(right_divisor_pair_list(n), int_pair_term_swapped(a, b)))
        sum(map(divisor_list(n), function(d: Nat) {
            a(d) * sum(map(divisor_list(divisor_quotient(n, d)), b))
        })) = sum(map(right_divisor_pair_list(n), int_pair_term_swapped(a, b)))
        swapped_right_pair_list_unique(n)
        swapped_right_pair_list(n).is_unique
        forall(x: Pair[Nat, Nat]) {
            swapped_right_pair_list_same_contains(n, x)
            swapped_right_pair_list(n).contains(x) = right_divisor_pair_list(n).contains(x)
        }
        unique_same_contains_map_sum_eq(swapped_right_pair_list(n), right_divisor_pair_list(n),
            int_pair_term_swapped(a, b))
        sum(map(swapped_right_pair_list(n), int_pair_term_swapped(a, b))) =
            sum(map(right_divisor_pair_list(n), int_pair_term_swapped(a, b)))
        sum(map(divisor_list(n), function(d: Nat) {
            a(d) * sum(map(divisor_list(divisor_quotient(n, d)), b))
        })) = sum(map(swapped_right_pair_list(n), int_pair_term_swapped(a, b)))
        swapped_list_reindex_sum(a, b, n)
        sum(map(swapped_right_pair_list(n), int_pair_term_swapped(a, b))) =
            sum(map(right_divisor_pair_list(n), int_pair_term(b, a)))
        sum(map(divisor_list(n), function(d: Nat) {
            a(d) * sum(map(divisor_list(divisor_quotient(n, d)), b))
        })) = sum(map(right_divisor_pair_list(n), int_pair_term(b, a)))
        int_right_expand(b, a, n)
        sum(map(divisor_list(n), function(e: Nat) {
            b(e) * sum(map(divisor_list(divisor_quotient(n, e)), a))
        })) = sum(map(right_divisor_pair_list(n), int_pair_term(b, a)))
        sum(map(divisor_list(n), function(d: Nat) {
            a(d) * sum(map(divisor_list(divisor_quotient(n, d)), b))
        })) = sum(map(divisor_list(n), function(e: Nat) {
            b(e) * sum(map(divisor_list(divisor_quotient(n, e)), a))
        }))
    }
    Nat.0 < n implies sum(map(divisor_list(n), function(d: Nat) {
        a(d) * sum(map(divisor_list(divisor_quotient(n, d)), b))
    })) =
        sum(map(divisor_list(n), function(e: Nat) {
            b(e) * sum(map(divisor_list(divisor_quotient(n, e)), a))
        }))
}

/// The self-indicator at `n`: 1 at the divisor `e` with `n / e = 1` (i.e. at
/// `e = n`) and 0 elsewhere.
define int_self_indicator(n: Nat) -> (Nat -> Int) {
    function(e: Nat) { if divisor_quotient(n, e) = Nat.1 { Int.1 } else { Int.0 } }
}

/// The weighted self-indicator: `f(e)` times the self-indicator at `n`.
define int_weighted_self_indicator(f: Nat -> Int, n: Nat) -> (Nat -> Int) {
    function(e: Nat) { f(e) * int_self_indicator(n)(e) }
}

/// Below `n`, the weighted self-indicator sum over the bounded divisor list
/// vanishes, since every bounded divisor has a cofactor different from one.
theorem int_weighted_indicator_below(f: Nat -> Int, n: Nat, k: Nat) {
    Nat.0 < n and k < n implies
        sum(map(divisors_up_to(n, k), int_weighted_self_indicator(f, n))) = Int.0
} by {
    define pred(bound: Nat) -> Bool {
        Nat.0 < n and bound < n implies
            sum(map(divisors_up_to(n, bound), int_weighted_self_indicator(f, n))) = Int.0
    }
    divisors_up_to_zero(n)
    divisors_up_to(n, Nat.0) = List.nil[Nat]
    map(List.nil[Nat], int_weighted_self_indicator(f, n)) = List.nil[Int]
    sum(List.nil[Int]) = Int.0
    pred(Nat.0)
    forall(j: Nat) {
        if pred(j) {
            pred(j) = (Nat.0 < n and j < n implies
                sum(map(divisors_up_to(n, j), int_weighted_self_indicator(f, n))) = Int.0)
            if Nat.0 < n and j.suc < n {
                j < j.suc
                lt_trans(j, j.suc, n)
                j < n
                if j.suc.divides(n) {
                    divisors_up_to_suc_yes(n, j)
                    divisors_up_to(n, j.suc) = List.cons(j.suc, divisors_up_to(n, j))
                    if divisor_quotient(n, j.suc) = Nat.1 {
                        divisor_quotient_cofactor(n, j.suc)
                        j.suc * divisor_quotient(n, j.suc) = n
                        j.suc * Nat.1 = j.suc
                        j.suc * divisor_quotient(n, j.suc) = j.suc * Nat.1
                        j.suc = n
                        false
                    }
                    divisor_quotient(n, j.suc) != Nat.1
                    int_self_indicator(n)(j.suc) =
                        (if divisor_quotient(n, j.suc) = Nat.1 { Int.1 } else { Int.0 })
                    (if divisor_quotient(n, j.suc) = Nat.1 { Int.1 } else { Int.0 }) = Int.0
                    int_self_indicator(n)(j.suc) = Int.0
                    int_weighted_self_indicator(f, n)(j.suc) =
                        f(j.suc) * int_self_indicator(n)(j.suc)
                    int_weighted_self_indicator(f, n)(j.suc) = f(j.suc) * Int.0
                    f(j.suc) * Int.0 = Int.0
                    int_weighted_self_indicator(f, n)(j.suc) = Int.0
                    map(List.cons(j.suc, divisors_up_to(n, j)), int_weighted_self_indicator(f, n)) =
                        List.cons(int_weighted_self_indicator(f, n)(j.suc),
                            map(divisors_up_to(n, j), int_weighted_self_indicator(f, n)))
                    map(divisors_up_to(n, j.suc), int_weighted_self_indicator(f, n)) =
                        map(List.cons(j.suc, divisors_up_to(n, j)), int_weighted_self_indicator(f, n))
                    sum(map(divisors_up_to(n, j.suc), int_weighted_self_indicator(f, n))) =
                        sum(List.cons(int_weighted_self_indicator(f, n)(j.suc),
                            map(divisors_up_to(n, j), int_weighted_self_indicator(f, n))))
                    sum(List.cons(int_weighted_self_indicator(f, n)(j.suc),
                            map(divisors_up_to(n, j), int_weighted_self_indicator(f, n)))) =
                        int_weighted_self_indicator(f, n)(j.suc) +
                            sum(map(divisors_up_to(n, j), int_weighted_self_indicator(f, n)))
                    pred(j)
                    sum(map(divisors_up_to(n, j), int_weighted_self_indicator(f, n))) = Int.0
                    sum(map(divisors_up_to(n, j.suc), int_weighted_self_indicator(f, n))) =
                        Int.0 + Int.0
                    sum(map(divisors_up_to(n, j.suc), int_weighted_self_indicator(f, n))) = Int.0
                } else {
                    divisors_up_to_suc_no(n, j)
                    divisors_up_to(n, j.suc) = divisors_up_to(n, j)
                    pred(j)
                    sum(map(divisors_up_to(n, j), int_weighted_self_indicator(f, n))) = Int.0
                    sum(map(divisors_up_to(n, j.suc), int_weighted_self_indicator(f, n))) = Int.0
                }
                sum(map(divisors_up_to(n, j.suc), int_weighted_self_indicator(f, n))) = Int.0
            }
            pred(j.suc) = (Nat.0 < n and j.suc < n implies
                sum(map(divisors_up_to(n, j.suc), int_weighted_self_indicator(f, n))) = Int.0)
            pred(j.suc)
        }
    }
    forall(bound: Nat) { pred(bound) implies pred(bound.suc) }
    pred(Nat.0) and forall(bound: Nat) { pred(bound) implies pred(bound.suc) }
    alt_induction(pred)
    forall(bound: Nat) { pred(bound) }
    pred(k)
    pred(k) = (Nat.0 < n and k < n implies
        sum(map(divisors_up_to(n, k), int_weighted_self_indicator(f, n))) = Int.0)
}

/// The weighted self-indicator sum over the full divisor list of positive `n`
/// isolates the top divisor: it equals `f(n)`.
theorem int_weighted_indicator_full(f: Nat -> Int, n: Nat) {
    Nat.0 < n implies
        sum(map(divisor_list(n), int_weighted_self_indicator(f, n))) = f(n)
} by {
    if Nat.0 < n {
        let m: Nat satisfy { m.suc = n }
        m.suc = n
        divides_self(n)
        n.divides(n)
        m.suc.divides(n)
        divisors_up_to_suc_yes(n, m)
        divisors_up_to(n, m.suc) = List.cons(m.suc, divisors_up_to(n, m))
        divisor_list(n) = divisors_up_to(n, n)
        divisor_list(n) = List.cons(n, divisors_up_to(n, m))
        map(List.cons(n, divisors_up_to(n, m)), int_weighted_self_indicator(f, n)) =
            List.cons(int_weighted_self_indicator(f, n)(n),
                map(divisors_up_to(n, m), int_weighted_self_indicator(f, n)))
        map(divisor_list(n), int_weighted_self_indicator(f, n)) =
            List.cons(int_weighted_self_indicator(f, n)(n),
                map(divisors_up_to(n, m), int_weighted_self_indicator(f, n)))
        sum(map(divisor_list(n), int_weighted_self_indicator(f, n))) =
            sum(List.cons(int_weighted_self_indicator(f, n)(n),
                map(divisors_up_to(n, m), int_weighted_self_indicator(f, n))))
        sum(List.cons(int_weighted_self_indicator(f, n)(n),
                map(divisors_up_to(n, m), int_weighted_self_indicator(f, n)))) =
            int_weighted_self_indicator(f, n)(n) +
                sum(map(divisors_up_to(n, m), int_weighted_self_indicator(f, n)))
        sum(map(divisor_list(n), int_weighted_self_indicator(f, n))) =
            int_weighted_self_indicator(f, n)(n) +
                sum(map(divisors_up_to(n, m), int_weighted_self_indicator(f, n)))
        divisor_quotient_self(n)
        divisor_quotient(n, n) = Nat.1
        int_self_indicator(n)(n) =
            (if divisor_quotient(n, n) = Nat.1 { Int.1 } else { Int.0 })
        (if divisor_quotient(n, n) = Nat.1 { Int.1 } else { Int.0 }) = Int.1
        int_self_indicator(n)(n) = Int.1
        int_weighted_self_indicator(f, n)(n) = f(n) * int_self_indicator(n)(n)
        int_weighted_self_indicator(f, n)(n) = f(n) * Int.1
        f(n) * Int.1 = f(n)
        int_weighted_self_indicator(f, n)(n) = f(n)
        m < m.suc
        m < n
        int_weighted_indicator_below(f, n, m)
        sum(map(divisors_up_to(n, m), int_weighted_self_indicator(f, n))) = Int.0
        sum(map(divisor_list(n), int_weighted_self_indicator(f, n))) = f(n) + Int.0
        f(n) + Int.0 = f(n)
        sum(map(divisor_list(n), int_weighted_self_indicator(f, n))) = f(n)
    }
}

/// The divisor sum of the Möbius function isolates the top divisor of `n`:
/// `sum_{e | n} f(e) * (sum_{d | n/e} mu(d)) = f(n)` for positive `n`.
theorem mobius_extracts_self(f: Nat -> Int, n: Nat) {
    Nat.0 < n implies
        sum(map(divisor_list(n), function(e: Nat) {
            f(e) * sum(map(divisor_list(divisor_quotient(n, e)), nat_mobius))
        })) = f(n)
} by {
    if Nat.0 < n {
        forall(e: Nat) {
            if divisor_list(n).contains(e) {
                nat_mobius_divisor_sum(divisor_quotient(n, e))
                sum(map(divisor_list(divisor_quotient(n, e)), nat_mobius)) =
                    (if divisor_quotient(n, e) = Nat.1 { Int.1 } else { Int.0 })
                int_self_indicator(n)(e) =
                    (if divisor_quotient(n, e) = Nat.1 { Int.1 } else { Int.0 })
                sum(map(divisor_list(divisor_quotient(n, e)), nat_mobius)) =
                    int_self_indicator(n)(e)
                f(e) * sum(map(divisor_list(divisor_quotient(n, e)), nat_mobius)) =
                    f(e) * int_self_indicator(n)(e)
                int_weighted_self_indicator(f, n)(e) = f(e) * int_self_indicator(n)(e)
                f(e) * sum(map(divisor_list(divisor_quotient(n, e)), nat_mobius)) =
                    int_weighted_self_indicator(f, n)(e)
            }
            divisor_list(n).contains(e) implies f(e) * sum(map(divisor_list(divisor_quotient(n, e)), nat_mobius)) = int_weighted_self_indicator(f, n)(e)
        }
        sum_map_of_pointwise(divisor_list(n),
            function(e: Nat) {
                f(e) * sum(map(divisor_list(divisor_quotient(n, e)), nat_mobius))
            },
            int_weighted_self_indicator(f, n))
        sum(map(divisor_list(n), function(e: Nat) {
            f(e) * sum(map(divisor_list(divisor_quotient(n, e)), nat_mobius))
        })) = sum(map(divisor_list(n), int_weighted_self_indicator(f, n)))
        int_weighted_indicator_full(f, n)
        sum(map(divisor_list(n), int_weighted_self_indicator(f, n))) = f(n)
        sum(map(divisor_list(n), function(e: Nat) {
            f(e) * sum(map(divisor_list(divisor_quotient(n, e)), nat_mobius))
        })) = f(n)
    }
    Nat.0 < n implies sum(map(divisor_list(n), function(e: Nat) {
        f(e) * sum(map(divisor_list(divisor_quotient(n, e)), nat_mobius))
    })) = f(n)
}

/// Möbius inversion, composition form: substituting the divisor sum of `f`
/// into the Möbius convolution recovers `f` itself:
/// `sum_{d | n} mu(d) * sum_{e | n/d} f(e) = f(n)` for positive `n`.
theorem mobius_inversion_composition(f: Nat -> Int, n: Nat) {
    Nat.0 < n implies
        sum(map(divisor_list(n), function(d: Nat) {
            nat_mobius(d) * sum(map(divisor_list(divisor_quotient(n, d)), f))
        })) = f(n)
} by {
    if Nat.0 < n {
        int_divisor_sum_interchange(nat_mobius, f, n)
        sum(map(divisor_list(n), function(d: Nat) {
            nat_mobius(d) * sum(map(divisor_list(divisor_quotient(n, d)), f))
        })) = sum(map(divisor_list(n), function(e: Nat) {
            f(e) * sum(map(divisor_list(divisor_quotient(n, e)), nat_mobius))
        }))
        mobius_extracts_self(f, n)
        sum(map(divisor_list(n), function(e: Nat) {
            f(e) * sum(map(divisor_list(divisor_quotient(n, e)), nat_mobius))
        })) = f(n)
        sum(map(divisor_list(n), function(d: Nat) {
            nat_mobius(d) * sum(map(divisor_list(divisor_quotient(n, d)), f))
        })) = f(n)
    }
    Nat.0 < n implies sum(map(divisor_list(n), function(d: Nat) {
        nat_mobius(d) * sum(map(divisor_list(divisor_quotient(n, d)), f))
    })) = f(n)
}

/// The integer-valued divisor-sum operator: `(D f)(n) = sum_{d | n} f(d)`.
/// This is the integer-valued companion of `divisor_sum_fn` from
/// `divisor_sum.ac`, which is defined only for natural-valued functions.
define int_divisor_sum_fn(f: Nat -> Int) -> (Nat -> Int) {
    function(n: Nat) { sum(map(divisor_list(n), f)) }
}

/// Möbius inversion: if `g(n) = sum_{d | n} f(d)` for every `n`, then
/// `f(n) = sum_{d | n} mu(d) * g(n / d)` for positive `n`.
theorem mobius_inversion(f: Nat -> Int, g: Nat -> Int, n: Nat) {
    Nat.0 < n and (forall(m: Nat) { g(m) = sum(map(divisor_list(m), f)) }) implies
        sum(map(divisor_list(n), function(d: Nat) {
            nat_mobius(d) * g(divisor_quotient(n, d))
        })) = f(n)
} by {
    if Nat.0 < n and (forall(m: Nat) { g(m) = sum(map(divisor_list(m), f)) }) {
        forall(d: Nat) {
            if divisor_list(n).contains(d) {
                g(divisor_quotient(n, d)) = sum(map(divisor_list(divisor_quotient(n, d)), f))
                nat_mobius(d) * g(divisor_quotient(n, d)) =
                    nat_mobius(d) * sum(map(divisor_list(divisor_quotient(n, d)), f))
            }
            divisor_list(n).contains(d) implies nat_mobius(d) * g(divisor_quotient(n, d)) =
                nat_mobius(d) * sum(map(divisor_list(divisor_quotient(n, d)), f))
        }
        sum_map_of_pointwise(divisor_list(n),
            function(d: Nat) { nat_mobius(d) * g(divisor_quotient(n, d)) },
            function(d: Nat) {
                nat_mobius(d) * sum(map(divisor_list(divisor_quotient(n, d)), f))
            })
        sum(map(divisor_list(n), function(d: Nat) {
            nat_mobius(d) * g(divisor_quotient(n, d))
        })) = sum(map(divisor_list(n), function(d: Nat) {
            nat_mobius(d) * sum(map(divisor_list(divisor_quotient(n, d)), f))
        }))
        mobius_inversion_composition(f, n)
        sum(map(divisor_list(n), function(d: Nat) {
            nat_mobius(d) * sum(map(divisor_list(divisor_quotient(n, d)), f))
        })) = f(n)
        sum(map(divisor_list(n), function(d: Nat) {
            nat_mobius(d) * g(divisor_quotient(n, d))
        })) = f(n)
    }
    Nat.0 < n and (forall(m: Nat) { g(m) = sum(map(divisor_list(m), f)) }) implies sum(map(divisor_list(n), function(d: Nat) {
        nat_mobius(d) * g(divisor_quotient(n, d))
    })) = f(n)
}

/// Möbius inversion, in terms of the divisor-sum operator: if `g` is the
/// divisor sum of `f`, then the Möbius convolution of `g` recovers `f`:
/// `f(n) = sum_{d | n} mu(d) * g(n / d)` for positive `n`.
theorem mobius_inversion_divisor_sum(f: Nat -> Int, g: Nat -> Int, n: Nat) {
    Nat.0 < n and (forall(m: Nat) { g(m) = int_divisor_sum_fn(f)(m) }) implies
        sum(map(divisor_list(n), function(d: Nat) {
            nat_mobius(d) * g(divisor_quotient(n, d))
        })) = f(n)
} by {
    if Nat.0 < n and (forall(m: Nat) { g(m) = int_divisor_sum_fn(f)(m) }) {
        forall(m: Nat) {
            int_divisor_sum_fn(f)(m) = sum(map(divisor_list(m), f))
            g(m) = int_divisor_sum_fn(f)(m)
            g(m) = sum(map(divisor_list(m), f))
        }
        Nat.0 < n and (forall(m: Nat) { g(m) = sum(map(divisor_list(m), f)) })
        mobius_inversion(f, g, n)
        sum(map(divisor_list(n), function(d: Nat) {
            nat_mobius(d) * g(divisor_quotient(n, d))
        })) = f(n)
    }
    Nat.0 < n and (forall(m: Nat) { g(m) = int_divisor_sum_fn(f)(m) }) implies sum(map(divisor_list(n), function(d: Nat) {
        nat_mobius(d) * g(divisor_quotient(n, d))
    })) = f(n)
}
