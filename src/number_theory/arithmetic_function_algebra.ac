/// Arithmetic-function algebra: the classical multiplicative functions and
/// their Dirichlet-convolution identities, stated in the framework of
/// `number_theory.arithmetic_functions` (`is_multiplicative_nat_fn`) and
/// `number_theory.dirichlet` (`dirichlet_convolve`).
///
/// The divisor-sum `nat_sigma`, the divisor-count `nat_tau`, and Euler's
/// totient `nat_totient` are multiplicative: `is_multiplicative_nat_fn` for
/// each.  The multiplicativity of `nat_sigma` and `nat_tau` is proved in
/// `number_theory.dirichlet` and restated here; the multiplicativity of
/// `nat_totient` is derived from `nat_totient_mul_coprime` in
/// `number_theory.totient`.  Dirichlet convolution preserves multiplicative
/// arithmetic functions (proved in `number_theory.dirichlet`), and the
/// fundamental identity `1 * mu = delta` is recorded in convolution form for
/// integer-valued arithmetic functions.
from nat import Nat
from int import Int
from list import map, sum
from number_theory.arithmetic_functions import is_multiplicative_nat_fn,
    multiplicative_nat_fn_apply
from number_theory.divisor_sum import nat_sigma, nat_tau
from number_theory.dirichlet import nat_sigma_multiplicative, nat_tau_multiplicative,
    dirichlet_convolve_multiplicative, dirichlet_convolve, divisor_quotient
from number_theory.divisor_sum import divisor_list
from number_theory.totient import nat_totient, nat_totient_one, nat_totient_mul_coprime
from number_theory.mobius_inversion import nat_mobius
from number_theory.mobius_inversion_theorem import int_one_arithmetic_fn,
    int_dirichlet_unit_fn, one_convolve_mobius, mobius_convolve_one
from data.basic.functions import function_extensionality
numerals Nat

/// The sum-of-divisors function `sigma(n)` is multiplicative: on coprime
/// arguments `sigma(m * n) = sigma(m) * sigma(n)`.  This restates
/// `nat_sigma_multiplicative` from `number_theory.dirichlet`.
theorem nat_sigma_multiplicative_law {
    is_multiplicative_nat_fn(nat_sigma)
} by {
    nat_sigma_multiplicative
    is_multiplicative_nat_fn(nat_sigma)
}

/// The divisor-count function `tau(n)` is multiplicative: on coprime
/// arguments `tau(m * n) = tau(m) * tau(n)`.  This restates
/// `nat_tau_multiplicative` from `number_theory.dirichlet`.
theorem nat_tau_multiplicative_law {
    is_multiplicative_nat_fn(nat_tau)
} by {
    nat_tau_multiplicative
    is_multiplicative_nat_fn(nat_tau)
}

/// Euler's totient `phi(n)` is one at one, so its coprimality product law
/// gives the multiplicativity statement.
theorem nat_totient_multiplicative {
    is_multiplicative_nat_fn(nat_totient)
} by {
    nat_totient_one
    nat_totient(Nat.1) = Nat.1
    forall(a: Nat, b: Nat) {
        if a.coprime(b) {
            nat_totient_mul_coprime(a, b)
            nat_totient(a * b) = nat_totient(a) * nat_totient(b)
        }
    }
    is_multiplicative_nat_fn(nat_totient)
}

/// The sum-of-divisors function is multiplicative on one coprime pair.
/// This restates `nat_sigma_multiplicative` in coprime-argument form.
theorem nat_sigma_mul_coprime_law(m: Nat, n: Nat) {
    m.coprime(n) implies nat_sigma(m * n) = nat_sigma(m) * nat_sigma(n)
} by {
    if m.coprime(n) {
        nat_sigma_multiplicative
        is_multiplicative_nat_fn(nat_sigma)
        multiplicative_nat_fn_apply(nat_sigma, m, n)
        nat_sigma(m * n) = nat_sigma(m) * nat_sigma(n)
    }
}

/// The divisor-count function is multiplicative on one coprime pair.
/// This restates `nat_tau_multiplicative` in coprime-argument form.
theorem nat_tau_mul_coprime_law(m: Nat, n: Nat) {
    m.coprime(n) implies nat_tau(m * n) = nat_tau(m) * nat_tau(n)
} by {
    if m.coprime(n) {
        nat_tau_multiplicative
        is_multiplicative_nat_fn(nat_tau)
        multiplicative_nat_fn_apply(nat_tau, m, n)
        nat_tau(m * n) = nat_tau(m) * nat_tau(n)
    }
}

/// The Dirichlet convolution of multiplicative arithmetic functions is
/// multiplicative.  This restates `dirichlet_convolve_multiplicative` from
/// `number_theory.dirichlet`.
theorem nat_dirichlet_convolve_multiplicative(f: Nat -> Nat, g: Nat -> Nat) {
    is_multiplicative_nat_fn(f) and is_multiplicative_nat_fn(g)
        implies is_multiplicative_nat_fn(dirichlet_convolve(f, g))
} by {
    if is_multiplicative_nat_fn(f) and is_multiplicative_nat_fn(g) {
        dirichlet_convolve_multiplicative(f, g)
        is_multiplicative_nat_fn(dirichlet_convolve(f, g))
    }
}

/// The integer-valued Dirichlet convolution: `(f * g)(n) = sum_{d | n} f(d) * g(n / d)`
/// for integer-valued arithmetic functions.
define int_dirichlet_convolve(f: Nat -> Int, g: Nat -> Int) -> (Nat -> Int) {
    function(n: Nat) {
        sum(map(divisor_list(n), function(d: Nat) {
            f(d) * g(divisor_quotient(n, d))
        }))
    }
}

/// Application of the integer-valued Dirichlet convolution unfolds to the
/// divisor-list sum.
theorem int_dirichlet_convolve_apply(f: Nat -> Int, g: Nat -> Int, n: Nat) {
    int_dirichlet_convolve(f, g)(n) =
        sum(map(divisor_list(n), function(d: Nat) {
            f(d) * g(divisor_quotient(n, d))
        }))
}

/// `1 * mu = delta`: convolving the constant-one function with the Möbius
/// function gives the Dirichlet unit, in convolution form.
theorem int_one_convolve_mobius(n: Nat) {
    int_dirichlet_convolve(int_one_arithmetic_fn, nat_mobius)(n) =
        int_dirichlet_unit_fn(n)
} by {
    int_dirichlet_convolve_apply(int_one_arithmetic_fn, nat_mobius, n)
    int_dirichlet_convolve(int_one_arithmetic_fn, nat_mobius)(n) =
        sum(map(divisor_list(n), function(d: Nat) {
            int_one_arithmetic_fn(d) * nat_mobius(divisor_quotient(n, d))
        }))
    one_convolve_mobius(n)
    sum(map(divisor_list(n), function(d: Nat) {
        int_one_arithmetic_fn(d) * nat_mobius(divisor_quotient(n, d))
    })) = int_dirichlet_unit_fn(n)
    int_dirichlet_convolve(int_one_arithmetic_fn, nat_mobius)(n) =
        int_dirichlet_unit_fn(n)
}

/// `1 * mu = delta` as an equality of integer-valued arithmetic functions.
theorem int_one_convolve_mobius_fn {
    int_dirichlet_convolve(int_one_arithmetic_fn, nat_mobius) =
        int_dirichlet_unit_fn
} by {
    forall(n: Nat) {
        int_one_convolve_mobius(n)
        int_dirichlet_convolve(int_one_arithmetic_fn, nat_mobius)(n) =
            int_dirichlet_unit_fn(n)
    }
    function_extensionality(int_dirichlet_convolve(int_one_arithmetic_fn, nat_mobius),
        int_dirichlet_unit_fn)
    int_dirichlet_convolve(int_one_arithmetic_fn, nat_mobius) =
        int_dirichlet_unit_fn
}

/// `mu * 1 = delta`: convolving the Möbius function with the constant-one
/// function gives the Dirichlet unit, in convolution form.
theorem int_mobius_convolve_one(n: Nat) {
    int_dirichlet_convolve(nat_mobius, int_one_arithmetic_fn)(n) =
        int_dirichlet_unit_fn(n)
} by {
    int_dirichlet_convolve_apply(nat_mobius, int_one_arithmetic_fn, n)
    int_dirichlet_convolve(nat_mobius, int_one_arithmetic_fn)(n) =
        sum(map(divisor_list(n), function(d: Nat) {
            nat_mobius(d) * int_one_arithmetic_fn(divisor_quotient(n, d))
        }))
    mobius_convolve_one(n)
    sum(map(divisor_list(n), function(d: Nat) {
        nat_mobius(d) * int_one_arithmetic_fn(divisor_quotient(n, d))
    })) = int_dirichlet_unit_fn(n)
    int_dirichlet_convolve(nat_mobius, int_one_arithmetic_fn)(n) =
        int_dirichlet_unit_fn(n)
}

/// `mu * 1 = delta` as an equality of integer-valued arithmetic functions.
theorem int_mobius_convolve_one_fn {
    int_dirichlet_convolve(nat_mobius, int_one_arithmetic_fn) =
        int_dirichlet_unit_fn
} by {
    forall(n: Nat) {
        int_mobius_convolve_one(n)
        int_dirichlet_convolve(nat_mobius, int_one_arithmetic_fn)(n) =
            int_dirichlet_unit_fn(n)
    }
    function_extensionality(int_dirichlet_convolve(nat_mobius, int_one_arithmetic_fn),
        int_dirichlet_unit_fn)
    int_dirichlet_convolve(nat_mobius, int_one_arithmetic_fn) =
        int_dirichlet_unit_fn
}
