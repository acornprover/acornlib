/// Möbius inversion applications: the classical identities obtained by
/// inverting divisor-sum identities with the Möbius function.
///
/// The general inversion theorem `mobius_inversion` (from
/// `mobius_inversion_theorem.ac`) says: if `g(n) = sum_{d | n} f(d)` for every
/// `n`, then `f(n) = sum_{d | n} mu(d) * g(n / d)` for positive `n`.  This
/// module applies it to the classical divisor-sum identities:
///
///   (a) `sum_{d | n} totient(d) = n` (totient_sums.ac) inverted to give
///       `totient(n) = sum_{d | n} mu(d) * (n / d)`;
///   (b) `sum_{d | n} sigma(d) = sum_{d | n} d * tau(n / d)`
///       (divisor_identities.ac), inverted to give
///       `sigma(n) = sum_{d | n} mu(d) * (id * tau)(n / d)`;
///   (d) the Mertens function `M(n) = sum_{k <= n} mu(k)` and its small
///       values.
///
/// The identities take integer values: natural-valued functions are lifted to
/// the integers with `Int.from_nat`, and the lifting commutes with sums of
/// naturals (`int_sum_map_from_nat`).
from nat import Nat
from int import Int, add_from_nat, add_assoc
from list import List, map, sum, sum_map_of_pointwise
from number_theory.mobius_inversion import nat_mobius, mertens, mertens_zero,
    mertens_one, nat_mobius_zero, nat_mobius_one, nat_mobius_prime
from number_theory.mobius_inversion_theorem import mobius_inversion
from number_theory.totient import nat_totient
from number_theory.totient_sums import totient_divisor_sum_identity
from number_theory.divisor_sum import divisor_list, divisor_sum_fn,
    divisor_sum_fn_apply, nat_sigma, nat_tau
from number_theory.divisor_identities import nat_sigma_divisor_sum_eq_id_convolve_tau_sum
from number_theory.dirichlet import divisor_quotient, dirichlet_term,
    dirichlet_term_apply
from number_theory.arithmetic_functions import nat_identity_arithmetic_fn
from number_theory.carmichael import two_is_prime, three_is_prime
numerals Nat
numerals Int

/// The identity arithmetic function lifted to the integers.
define int_nat_identity_fn(n: Nat) -> Int {
    Int.from_nat(n)
}

/// Euler's totient lifted to the integers.
define int_nat_totient_fn(n: Nat) -> Int {
    Int.from_nat(nat_totient(n))
}

/// The divisor-sum function of sigma lifted to the integers.
define int_nat_sigma_fn(n: Nat) -> Int {
    Int.from_nat(nat_sigma(n))
}

/// The convolution `id * tau` of the identity with the divisor-count
/// function: `(id * tau)(n) = sum_{d | n} d * tau(n / d)`.
define id_times_tau(n: Nat) -> Nat {
    sum(map(divisor_list(n), function(d: Nat) { d * nat_tau(divisor_quotient(n, d)) }))
}

/// The convolution `id * tau` lifted to the integers.
define int_id_times_tau_fn(n: Nat) -> Int {
    Int.from_nat(id_times_tau(n))
}

/// The pointwise lifting of a natural-valued function to the integers.
define lift_from_nat(h: Nat -> Nat) -> (Nat -> Int) {
    function(x: Nat) { Int.from_nat(h(x)) }
}

/// Lifting a natural-valued sum to the integers commutes with `Int.from_nat`:
/// `sum_{x in l} Int.from_nat(h(x)) = Int.from_nat(sum_{x in l} h(x))`.
theorem int_sum_map_from_nat(l: List[Nat], h: Nat -> Nat) {
    sum(map(l, lift_from_nat(h))) = Int.from_nat(sum(map(l, h)))
} by {
    define p(ls: List[Nat]) -> Bool {
        sum(map(ls, lift_from_nat(h))) = Int.from_nat(sum(map(ls, h)))
    }
    map(List.nil[Nat], lift_from_nat(h)) = List.nil[Int]
    sum(List.nil[Int]) = Int.0
    map(List.nil[Nat], h) = List.nil[Nat]
    sum(List.nil[Nat]) = Nat.0
    Int.from_nat(Nat.0) = Int.0
    p(List.nil[Nat])
    forall(head: Nat, tail: List[Nat]) {
        if p(tail) {
            p(tail) = (sum(map(tail, lift_from_nat(h))) =
                Int.from_nat(sum(map(tail, h))))
            map(List.cons(head, tail), lift_from_nat(h)) =
                List.cons(lift_from_nat(h)(head), map(tail, lift_from_nat(h)))
            lift_from_nat(h)(head) = Int.from_nat(h(head))
            map(List.cons(head, tail), lift_from_nat(h)) =
                List.cons(Int.from_nat(h(head)), map(tail, lift_from_nat(h)))
            sum(map(List.cons(head, tail), lift_from_nat(h))) =
                sum(List.cons(Int.from_nat(h(head)), map(tail, lift_from_nat(h))))
            sum(List.cons(Int.from_nat(h(head)), map(tail, lift_from_nat(h)))) =
                Int.from_nat(h(head)) + sum(map(tail, lift_from_nat(h)))
            sum(map(List.cons(head, tail), lift_from_nat(h))) =
                Int.from_nat(h(head)) + sum(map(tail, lift_from_nat(h)))
            map(List.cons(head, tail), h) = List.cons(h(head), map(tail, h))
            sum(map(List.cons(head, tail), h)) = h(head) + sum(map(tail, h))
            add_from_nat(h(head), sum(map(tail, h)))
            Int.from_nat(h(head)) + Int.from_nat(sum(map(tail, h))) =
                Int.from_nat(h(head) + sum(map(tail, h)))
            Int.from_nat(sum(map(List.cons(head, tail), h))) =
                Int.from_nat(h(head) + sum(map(tail, h)))
            Int.from_nat(h(head)) + Int.from_nat(sum(map(tail, h))) =
                Int.from_nat(sum(map(List.cons(head, tail), h)))
            Int.from_nat(h(head)) + sum(map(tail, lift_from_nat(h))) =
                Int.from_nat(sum(map(List.cons(head, tail), h)))
            sum(map(List.cons(head, tail), lift_from_nat(h))) =
                Int.from_nat(sum(map(List.cons(head, tail), h)))
            p(List.cons(head, tail)) =
                (sum(map(List.cons(head, tail), lift_from_nat(h))) =
                    Int.from_nat(sum(map(List.cons(head, tail), h))))
            p(List.cons(head, tail))
        }
    }
    forall(head: Nat, tail: List[Nat]) {
        p(tail) implies p(List.cons(head, tail))
    }
    p(List.nil[Nat]) and forall(head: Nat, tail: List[Nat]) {
        p(tail) implies p(List.cons(head, tail))
    }
    List.induction(p)
    forall(ls: List[Nat]) { p(ls) }
    p(l)
}

// ---------------------------------------------------------------------------
// (a) Euler's totient by Möbius inversion.
//
// `sum_{d | n} totient(d) = n` (totient_divisor_sum_identity) is inverted at
// `f = Int.from_nat . totient` and `g = Int.from_nat . id`, giving
// `totient(n) = sum_{d | n} mu(d) * (n / d)`.
// ---------------------------------------------------------------------------

/// Euler's totient via Möbius inversion:
/// `totient(n) = sum_{d | n} mu(d) * (n / d)` for positive `n`.
theorem totient_mobius_inversion(n: Nat) {
    Nat.0 < n implies
        sum(map(divisor_list(n), function(d: Nat) {
            nat_mobius(d) * Int.from_nat(divisor_quotient(n, d))
        })) = Int.from_nat(nat_totient(n))
} by {
    if Nat.0 < n {
        forall(m: Nat) {
            totient_divisor_sum_identity(m)
            divisor_sum_fn(nat_totient)(m) = m
            divisor_sum_fn_apply(nat_totient, m)
            divisor_sum_fn(nat_totient)(m) = sum(map(divisor_list(m), nat_totient))
            sum(map(divisor_list(m), nat_totient)) = m
            int_sum_map_from_nat(divisor_list(m), nat_totient)
            sum(map(divisor_list(m), lift_from_nat(nat_totient))) =
                Int.from_nat(sum(map(divisor_list(m), nat_totient)))
            Int.from_nat(sum(map(divisor_list(m), nat_totient))) = Int.from_nat(m)
            sum(map(divisor_list(m), lift_from_nat(nat_totient))) =
                Int.from_nat(m)
            forall(x: Nat) {
                if divisor_list(m).contains(x) {
                    int_nat_totient_fn(x) = lift_from_nat(nat_totient)(x)
                }
                divisor_list(m).contains(x) implies int_nat_totient_fn(x) = lift_from_nat(nat_totient)(x)
            }
            sum_map_of_pointwise(divisor_list(m), int_nat_totient_fn, lift_from_nat(nat_totient))
            sum(map(divisor_list(m), int_nat_totient_fn)) =
                sum(map(divisor_list(m), lift_from_nat(nat_totient)))
            sum(map(divisor_list(m), int_nat_totient_fn)) = Int.from_nat(m)
            int_nat_identity_fn(m) = Int.from_nat(m)
            int_nat_identity_fn(m) = sum(map(divisor_list(m), int_nat_totient_fn))
        }
        Nat.0 < n and (forall(m: Nat) {
            int_nat_identity_fn(m) = sum(map(divisor_list(m), int_nat_totient_fn))
        })
        mobius_inversion(int_nat_totient_fn, int_nat_identity_fn, n)
        sum(map(divisor_list(n), function(d: Nat) {
            nat_mobius(d) * int_nat_identity_fn(divisor_quotient(n, d))
        })) = int_nat_totient_fn(n)
        forall(d: Nat) {
            if divisor_list(n).contains(d) {
                int_nat_identity_fn(divisor_quotient(n, d)) =
                    Int.from_nat(divisor_quotient(n, d))
                nat_mobius(d) * int_nat_identity_fn(divisor_quotient(n, d)) =
                    nat_mobius(d) * Int.from_nat(divisor_quotient(n, d))
            }
            divisor_list(n).contains(d) implies nat_mobius(d) * int_nat_identity_fn(divisor_quotient(n, d)) =
                nat_mobius(d) * Int.from_nat(divisor_quotient(n, d))
        }
        sum_map_of_pointwise(divisor_list(n),
            function(d: Nat) {
                nat_mobius(d) * int_nat_identity_fn(divisor_quotient(n, d))
            },
            function(d: Nat) {
                nat_mobius(d) * Int.from_nat(divisor_quotient(n, d))
            })
        sum(map(divisor_list(n), function(d: Nat) {
            nat_mobius(d) * int_nat_identity_fn(divisor_quotient(n, d))
        })) = sum(map(divisor_list(n), function(d: Nat) {
            nat_mobius(d) * Int.from_nat(divisor_quotient(n, d))
        }))
        int_nat_totient_fn(n) = Int.from_nat(nat_totient(n))
        sum(map(divisor_list(n), function(d: Nat) {
            nat_mobius(d) * Int.from_nat(divisor_quotient(n, d))
        })) = Int.from_nat(nat_totient(n))
    }
}

// ---------------------------------------------------------------------------
// (b) The sigma-tau convolution identity and its Möbius inversion.
//
// `sum_{d | n} sigma(d) = sum_{d | n} d * tau(n / d)` is the convolution
// identity `sigma * 1 = id * tau` (divisor_identities.ac).  Inverting it with
// `mobius_inversion` at `f = Int.from_nat . sigma` and
// `g = Int.from_nat . (id * tau)` gives
// `sigma(n) = sum_{d | n} mu(d) * (id * tau)(n / d)`.
//
// Note: the literal identity `sigma(n) = sum_{d | n} d * tau(n / d)` is
// false (at `n = 2`, `sigma(2) = 3` but `sum_{d | 2} d * tau(2 / d) = 4`);
// the correct classical statement is the divisor-sum form above.
// ---------------------------------------------------------------------------

/// The classical convolution identity for sigma and tau in the divisor-sum
/// form: `sum_{d | n} sigma(d) = sum_{d | n} d * tau(n / d)`.
theorem sigma_divisor_sum_tau_weighted(n: Nat) {
    divisor_sum_fn(nat_sigma)(n) = id_times_tau(n)
} by {
    nat_sigma_divisor_sum_eq_id_convolve_tau_sum(n)
    divisor_sum_fn(nat_sigma)(n) =
        sum(map(divisor_list(n), dirichlet_term(nat_identity_arithmetic_fn, nat_tau, n)))
    forall(d: Nat) {
        if divisor_list(n).contains(d) {
            dirichlet_term_apply(nat_identity_arithmetic_fn, nat_tau, n, d)
            dirichlet_term(nat_identity_arithmetic_fn, nat_tau, n)(d) =
                nat_identity_arithmetic_fn(d) * nat_tau(divisor_quotient(n, d))
            nat_identity_arithmetic_fn(d) = d
            dirichlet_term(nat_identity_arithmetic_fn, nat_tau, n)(d) =
                d * nat_tau(divisor_quotient(n, d))
        }
        divisor_list(n).contains(d) implies dirichlet_term(nat_identity_arithmetic_fn, nat_tau, n)(d) =
            d * nat_tau(divisor_quotient(n, d))
    }
    sum_map_of_pointwise(divisor_list(n),
        dirichlet_term(nat_identity_arithmetic_fn, nat_tau, n),
        function(d: Nat) { d * nat_tau(divisor_quotient(n, d)) })
    sum(map(divisor_list(n), dirichlet_term(nat_identity_arithmetic_fn, nat_tau, n))) =
        sum(map(divisor_list(n), function(d: Nat) { d * nat_tau(divisor_quotient(n, d)) }))
    id_times_tau(n) =
        sum(map(divisor_list(n), function(d: Nat) { d * nat_tau(divisor_quotient(n, d)) }))
    divisor_sum_fn(nat_sigma)(n) = id_times_tau(n)
}

/// Sigma by Möbius inversion of the sigma-tau identity:
/// `sigma(n) = sum_{d | n} mu(d) * (id * tau)(n / d)` for positive `n`.
theorem sigma_mobius_inversion(n: Nat) {
    Nat.0 < n implies
        sum(map(divisor_list(n), function(d: Nat) {
            nat_mobius(d) * int_id_times_tau_fn(divisor_quotient(n, d))
        })) = int_nat_sigma_fn(n)
} by {
    if Nat.0 < n {
        forall(m: Nat) {
            sigma_divisor_sum_tau_weighted(m)
            divisor_sum_fn(nat_sigma)(m) = id_times_tau(m)
            divisor_sum_fn_apply(nat_sigma, m)
            divisor_sum_fn(nat_sigma)(m) = sum(map(divisor_list(m), nat_sigma))
            sum(map(divisor_list(m), nat_sigma)) = id_times_tau(m)
            int_sum_map_from_nat(divisor_list(m), nat_sigma)
            sum(map(divisor_list(m), lift_from_nat(nat_sigma))) =
                Int.from_nat(sum(map(divisor_list(m), nat_sigma)))
            Int.from_nat(sum(map(divisor_list(m), nat_sigma))) =
                Int.from_nat(id_times_tau(m))
            sum(map(divisor_list(m), lift_from_nat(nat_sigma))) =
                Int.from_nat(id_times_tau(m))
            forall(x: Nat) {
                if divisor_list(m).contains(x) {
                    int_nat_sigma_fn(x) = lift_from_nat(nat_sigma)(x)
                }
                divisor_list(m).contains(x) implies int_nat_sigma_fn(x) = lift_from_nat(nat_sigma)(x)
            }
            sum_map_of_pointwise(divisor_list(m), int_nat_sigma_fn, lift_from_nat(nat_sigma))
            sum(map(divisor_list(m), int_nat_sigma_fn)) =
                sum(map(divisor_list(m), lift_from_nat(nat_sigma)))
            sum(map(divisor_list(m), int_nat_sigma_fn)) = Int.from_nat(id_times_tau(m))
            int_id_times_tau_fn(m) = Int.from_nat(id_times_tau(m))
            int_id_times_tau_fn(m) = sum(map(divisor_list(m), int_nat_sigma_fn))
        }
        Nat.0 < n and (forall(m: Nat) {
            int_id_times_tau_fn(m) = sum(map(divisor_list(m), int_nat_sigma_fn))
        })
        mobius_inversion(int_nat_sigma_fn, int_id_times_tau_fn, n)
        sum(map(divisor_list(n), function(d: Nat) {
            nat_mobius(d) * int_id_times_tau_fn(divisor_quotient(n, d))
        })) = int_nat_sigma_fn(n)
    }
}

// ---------------------------------------------------------------------------
// (d) The Mertens function.
//
// The Mertens function `M(n) = sum_{k <= n} mu(k)` is defined in
// `mobius_inversion.ac` (`mertens`); `M(0) = 0` and `M(1) = 1` are proved
// there.  This section records the definitional identity and the next small
// values.  The classical closed form
//
//     sum_{k <= n} totient(k) = (1 + sum_{k <= n} mu(k) * floor(n / k)^2) / 2
//
// relating the totient summatory function to Mertens-style sums needs floor
// division and rational arithmetic that the library does not yet provide, so
// it is left as a research target.
// ---------------------------------------------------------------------------

/// The Mertens function is the prefix sum of the Möbius function:
/// `M(n) = sum_{k <= n} mu(k)`.
theorem mertens_prefix_sum(n: Nat) {
    mertens(n) = sum(map(n.suc.range, nat_mobius))
} by {
    mertens(n) = sum(map(n.suc.range, nat_mobius))
}

/// Appending to a cons list appends to its tail:
/// `List.cons(a, l).append(x) = List.cons(a, l.append(x))`.
theorem cons_append[T](a: T, l: List[T], x: T) {
    List.cons(a, l).append(x) = List.cons(a, l.append(x))
} by {
    List.cons(a, l).append(x) = List.cons(a, l) + List.singleton(x)
    List.cons(a, l) + List.singleton(x) = List.cons(a, l + List.singleton(x))
    l + List.singleton(x) = l.append(x)
    List.cons(a, l + List.singleton(x)) = List.cons(a, l.append(x))
    List.cons(a, l).append(x) = List.cons(a, l.append(x))
}

/// `M(2) = mu(0) + mu(1) + mu(2) = 0`.
theorem mertens_two {
    mertens(Nat.2) = Int.0
} by {
    mertens(Nat.2) = sum(map(Nat.2.suc.range, nat_mobius))
    Nat.2.suc = Nat.3
    Nat.0.range = List.nil[Nat]
    Nat.1.range = Nat.0.range.append(Nat.0)
    Nat.1.range = List.cons(Nat.0, List.nil[Nat])
    Nat.2.range = Nat.1.range.append(Nat.1)
    List.cons(Nat.0, List.nil[Nat]).append(Nat.1) =
        List.cons(Nat.0, List.cons(Nat.1, List.nil[Nat]))
    Nat.2.range = List.cons(Nat.0, List.cons(Nat.1, List.nil[Nat]))
    Nat.3.range = Nat.2.range.append(Nat.2)
    cons_append(Nat.0, List.cons(Nat.1, List.nil[Nat]), Nat.2)
    List.cons(Nat.0, List.cons(Nat.1, List.nil[Nat])).append(Nat.2) =
        List.cons(Nat.0, List.cons(Nat.1, List.nil[Nat]).append(Nat.2))
    cons_append(Nat.1, List.nil[Nat], Nat.2)
    List.cons(Nat.1, List.nil[Nat]).append(Nat.2) =
        List.cons(Nat.1, List.nil[Nat].append(Nat.2))
    List.nil[Nat].append(Nat.2) = List.cons(Nat.2, List.nil[Nat])
    List.cons(Nat.1, List.nil[Nat].append(Nat.2)) =
        List.cons(Nat.1, List.cons(Nat.2, List.nil[Nat]))
    List.cons(Nat.0, List.cons(Nat.1, List.nil[Nat]).append(Nat.2)) =
        List.cons(Nat.0, List.cons(Nat.1, List.cons(Nat.2, List.nil[Nat])))
    List.cons(Nat.0, List.cons(Nat.1, List.nil[Nat])).append(Nat.2) =
        List.cons(Nat.0, List.cons(Nat.1, List.cons(Nat.2, List.nil[Nat])))
    Nat.3.range = List.cons(Nat.0, List.cons(Nat.1, List.cons(Nat.2, List.nil[Nat])))
    map(Nat.3.range, nat_mobius) =
        map(List.cons(Nat.0, List.cons(Nat.1, List.cons(Nat.2, List.nil[Nat]))), nat_mobius)
    map(List.cons(Nat.0, List.cons(Nat.1, List.cons(Nat.2, List.nil[Nat]))), nat_mobius) =
        List.cons(nat_mobius(Nat.0),
            map(List.cons(Nat.1, List.cons(Nat.2, List.nil[Nat])), nat_mobius))
    map(List.cons(Nat.1, List.cons(Nat.2, List.nil[Nat])), nat_mobius) =
        List.cons(nat_mobius(Nat.1), map(List.cons(Nat.2, List.nil[Nat]), nat_mobius))
    map(List.cons(Nat.2, List.nil[Nat]), nat_mobius) =
        List.cons(nat_mobius(Nat.2), map(List.nil[Nat], nat_mobius))
    map(List.nil[Nat], nat_mobius) = List.nil[Int]
    nat_mobius_zero
    nat_mobius(Nat.0) = Int.0
    nat_mobius_one
    nat_mobius(Nat.1) = Int.1
    two_is_prime
    Nat.2.is_prime
    nat_mobius_prime(Nat.2)
    nat_mobius(Nat.2) = -Int.1
    sum(List.cons(Int.0, List.cons(Int.1, List.cons(-Int.1, List.nil[Int])))) =
        Int.0 + sum(List.cons(Int.1, List.cons(-Int.1, List.nil[Int])))
    sum(List.cons(Int.1, List.cons(-Int.1, List.nil[Int]))) =
        Int.1 + sum(List.cons(-Int.1, List.nil[Int]))
    sum(List.cons(-Int.1, List.nil[Int])) = -Int.1 + sum(List.nil[Int])
    sum(List.nil[Int]) = Int.0
    -Int.1 + Int.0 = -Int.1
    Int.1 + -Int.1 = Int.0
    Int.0 + Int.0 = Int.0
    sum(map(Nat.3.range, nat_mobius)) = Int.0
    mertens(Nat.2) = Int.0
}

/// `M(3) = mu(0) + mu(1) + mu(2) + mu(3) = -1`.
theorem mertens_three {
    mertens(Nat.3) = -Int.1
} by {
    mertens(Nat.3) = sum(map(Nat.3.suc.range, nat_mobius))
    Nat.3.suc = Nat.4
    Nat.0.range = List.nil[Nat]
    Nat.1.range = Nat.0.range.append(Nat.0)
    Nat.1.range = List.cons(Nat.0, List.nil[Nat])
    Nat.2.range = Nat.1.range.append(Nat.1)
    List.cons(Nat.0, List.nil[Nat]).append(Nat.1) =
        List.cons(Nat.0, List.cons(Nat.1, List.nil[Nat]))
    Nat.2.range = List.cons(Nat.0, List.cons(Nat.1, List.nil[Nat]))
    Nat.3.range = Nat.2.range.append(Nat.2)
    cons_append(Nat.0, List.cons(Nat.1, List.nil[Nat]), Nat.2)
    List.cons(Nat.0, List.cons(Nat.1, List.nil[Nat])).append(Nat.2) =
        List.cons(Nat.0, List.cons(Nat.1, List.nil[Nat]).append(Nat.2))
    cons_append(Nat.1, List.nil[Nat], Nat.2)
    List.cons(Nat.1, List.nil[Nat]).append(Nat.2) =
        List.cons(Nat.1, List.nil[Nat].append(Nat.2))
    List.nil[Nat].append(Nat.2) = List.cons(Nat.2, List.nil[Nat])
    List.cons(Nat.1, List.nil[Nat].append(Nat.2)) =
        List.cons(Nat.1, List.cons(Nat.2, List.nil[Nat]))
    List.cons(Nat.0, List.cons(Nat.1, List.nil[Nat]).append(Nat.2)) =
        List.cons(Nat.0, List.cons(Nat.1, List.cons(Nat.2, List.nil[Nat])))
    List.cons(Nat.0, List.cons(Nat.1, List.nil[Nat])).append(Nat.2) =
        List.cons(Nat.0, List.cons(Nat.1, List.cons(Nat.2, List.nil[Nat])))
    Nat.3.range = List.cons(Nat.0, List.cons(Nat.1, List.cons(Nat.2, List.nil[Nat])))
    Nat.4.range = Nat.3.range.append(Nat.3)
    cons_append(Nat.0, List.cons(Nat.1, List.cons(Nat.2, List.nil[Nat])), Nat.3)
    List.cons(Nat.0, List.cons(Nat.1, List.cons(Nat.2, List.nil[Nat]))).append(Nat.3) =
        List.cons(Nat.0, List.cons(Nat.1, List.cons(Nat.2, List.nil[Nat])).append(Nat.3))
    cons_append(Nat.1, List.cons(Nat.2, List.nil[Nat]), Nat.3)
    List.cons(Nat.1, List.cons(Nat.2, List.nil[Nat])).append(Nat.3) =
        List.cons(Nat.1, List.cons(Nat.2, List.nil[Nat]).append(Nat.3))
    cons_append(Nat.2, List.nil[Nat], Nat.3)
    List.cons(Nat.2, List.nil[Nat]).append(Nat.3) =
        List.cons(Nat.2, List.nil[Nat].append(Nat.3))
    List.nil[Nat].append(Nat.3) = List.cons(Nat.3, List.nil[Nat])
    List.cons(Nat.2, List.nil[Nat].append(Nat.3)) =
        List.cons(Nat.2, List.cons(Nat.3, List.nil[Nat]))
    List.cons(Nat.1, List.cons(Nat.2, List.nil[Nat]).append(Nat.3)) =
        List.cons(Nat.1, List.cons(Nat.2, List.cons(Nat.3, List.nil[Nat])))
    List.cons(Nat.0, List.cons(Nat.1, List.cons(Nat.2, List.nil[Nat])).append(Nat.3)) =
        List.cons(Nat.0, List.cons(Nat.1, List.cons(Nat.2, List.cons(Nat.3, List.nil[Nat]))))
    List.cons(Nat.0, List.cons(Nat.1, List.cons(Nat.2, List.nil[Nat]))).append(Nat.3) =
        List.cons(Nat.0, List.cons(Nat.1, List.cons(Nat.2, List.cons(Nat.3, List.nil[Nat]))))
    Nat.4.range =
        List.cons(Nat.0, List.cons(Nat.1, List.cons(Nat.2, List.cons(Nat.3, List.nil[Nat]))))
    map(Nat.4.range, nat_mobius) =
        map(List.cons(Nat.0, List.cons(Nat.1, List.cons(Nat.2, List.cons(Nat.3, List.nil[Nat])))), nat_mobius)
    map(List.cons(Nat.0, List.cons(Nat.1, List.cons(Nat.2, List.cons(Nat.3, List.nil[Nat])))), nat_mobius) =
        List.cons(nat_mobius(Nat.0),
            map(List.cons(Nat.1, List.cons(Nat.2, List.cons(Nat.3, List.nil[Nat]))), nat_mobius))
    map(List.cons(Nat.1, List.cons(Nat.2, List.cons(Nat.3, List.nil[Nat]))), nat_mobius) =
        List.cons(nat_mobius(Nat.1),
            map(List.cons(Nat.2, List.cons(Nat.3, List.nil[Nat])), nat_mobius))
    map(List.cons(Nat.2, List.cons(Nat.3, List.nil[Nat])), nat_mobius) =
        List.cons(nat_mobius(Nat.2), map(List.cons(Nat.3, List.nil[Nat]), nat_mobius))
    map(List.cons(Nat.3, List.nil[Nat]), nat_mobius) =
        List.cons(nat_mobius(Nat.3), map(List.nil[Nat], nat_mobius))
    map(List.nil[Nat], nat_mobius) = List.nil[Int]
    nat_mobius_zero
    nat_mobius(Nat.0) = Int.0
    nat_mobius_one
    nat_mobius(Nat.1) = Int.1
    two_is_prime
    Nat.2.is_prime
    nat_mobius_prime(Nat.2)
    nat_mobius(Nat.2) = -Int.1
    three_is_prime
    Nat.3.is_prime
    nat_mobius_prime(Nat.3)
    nat_mobius(Nat.3) = -Int.1
    sum(List.cons(Int.0, List.cons(Int.1, List.cons(-Int.1, List.cons(-Int.1, List.nil[Int]))))) =
        Int.0 + sum(List.cons(Int.1, List.cons(-Int.1, List.cons(-Int.1, List.nil[Int]))))
    sum(List.cons(Int.1, List.cons(-Int.1, List.cons(-Int.1, List.nil[Int])))) =
        Int.1 + sum(List.cons(-Int.1, List.cons(-Int.1, List.nil[Int])))
    sum(List.cons(-Int.1, List.cons(-Int.1, List.nil[Int]))) =
        -Int.1 + sum(List.cons(-Int.1, List.nil[Int]))
    sum(List.cons(-Int.1, List.nil[Int])) = -Int.1 + sum(List.nil[Int])
    sum(List.nil[Int]) = Int.0
    -Int.1 + Int.0 = -Int.1
    sum(List.cons(-Int.1, List.cons(-Int.1, List.nil[Int]))) = -Int.1 + -Int.1
    add_assoc(Int.1, -Int.1, -Int.1)
    Int.1 + (-Int.1 + -Int.1) = (Int.1 + -Int.1) + -Int.1
    Int.1 + -Int.1 = Int.0
    (Int.1 + -Int.1) + -Int.1 = Int.0 + -Int.1
    Int.0 + -Int.1 = -Int.1
    (Int.1 + -Int.1) + -Int.1 = -Int.1
    Int.1 + (-Int.1 + -Int.1) = -Int.1
    sum(List.cons(Int.1, List.cons(-Int.1, List.cons(-Int.1, List.nil[Int])))) = -Int.1
    Int.0 + -Int.1 = -Int.1
    sum(List.cons(Int.0, List.cons(Int.1, List.cons(-Int.1, List.cons(-Int.1, List.nil[Int]))))) = -Int.1
    sum(map(Nat.4.range, nat_mobius)) = -Int.1
    mertens(Nat.3) = -Int.1
}

// ---------------------------------------------------------------------------
// (c) Counting primitive Pythagorean triples by Möbius inversion.
//
// The classical application counts the primitive Pythagorean triples with
// hypotenuse at most `N`: every primitive triple is `(m^2 - n^2, 2 m n,
// m^2 + n^2)` with coprime `m > n` of opposite parity, so the number of
// primitive triples with hypotenuse at most `N` is the count of coprime pairs
// `(m, n)`, `m > n`, `m^2 + n^2 <= N`, of opposite parity.  Counting the pairs
// without the coprimality condition by parity and inverting with the Möbius
// function gives the exact count.  The library has no Pythagorean-triple or
// coprime-pair-counting API yet (only the Pythagorean theorem itself in
// `top100`), so this result is left as a research target.
// ---------------------------------------------------------------------------
