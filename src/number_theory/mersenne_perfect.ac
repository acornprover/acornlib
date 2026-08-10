// Mersenne primes and perfect numbers.
//
// A Mersenne number is `2^p - 1`; a Mersenne prime is a Mersenne number that
// is prime.  The classical facts proved here are:
//
//   (a) if `2^p - 1` is prime then `p` is prime (the necessity direction:
//       `p = a b` with `1 < a` forces `2^a - 1` to divide `2^p - 1`);
//   (b) `2^2 - 1 = 3`, `2^3 - 1 = 7` and `2^5 - 1 = 31` are prime;
//   (c) `2^11 - 1 = 2047 = 23 * 89` is the first composite Mersenne number;
//   (d) Euclid's construction: if `2^p - 1` is prime then
//       `2^(p-1) (2^p - 1)` is a perfect number (via the multiplicativity of
//       `sigma` on the coprime factors `2^(p-1)` and `2^p - 1`).
//
// The converse direction of (d) — Euler's theorem that every even perfect
// number has this form — is stated at the end of the file with a proof
// sketch, but not formalised.

from nat import Nat
from nat import divides_trans, divides_mul, divides_sub, add_sub,
    add_imp_sub, add_assoc, add_comm, add_zero_right, add_one_right,
    add_one_left, mul_comm, mul_assoc, mul_two_left, mul_one_left,
    mul_one_right, mul_zero_right, lte_mul, lt_mul_both, lte_mul_both,
    lte_add_left, exp_add, exp_mul, exp_one, exp_zero, exp_ne_zero,
    lt_trans, lt_suc_right, lt_suc, not_lt_zero, lt_not_ref,
    lte_imp_not_lt, lt_imp_lte_suc, lt_and_lte, lte_and_lt,
    only_zero_lte_zero, suc_sub_one, sub_self, zero_or_suc, pos_of_ne_zero,
    lt_or_lte, lte_trans, gcd_divides_left, gcd_divides_right, divisor_lt,
    read_add_single, read_add_read, read_mul_single, read_read_carry,
    nat_mul_1_3, nat_mul_2_2, nat_mul_2_8, nat_mul_2_9, nat_mul_3_3,
    nat_mul_3_8, nat_mul_3_9, nat_mul_5_6, nat_mul_6_6, nat_mul_7_5
from list import List, map, sum, partial, partial_split_last
from number_theory.zsigmondy import two_pow_two, two_pow_three, two_pow_four,
    lt_zero_one, lt_one_two, lt_zero_two, not_divides_of_lt,
    divides_suc_pair_imp_one, lt_ne, one_le_two_pow, two_divides_two_pow,
    two_pow_sub_one_gt_one, seven_is_prime
from number_theory.sigma_multiplicative import sub_one_add_one,
    nat_sigma_prime_pow_mult, nat_sigma_mul_coprime
from number_theory.divisor_sum import nat_sigma, nat_sigma_prime
from number_theory.factorisation import no_proper_divisor_imp_prime,
    prime_divisor_is_one_or_self
from number_theory.perfect_numbers import is_perfect
from data.nat.nat_binary_digits import two_pow_positive
numerals Nat

// ---------------------------------------------------------------------------
// Decimal arithmetic, built explicitly with the base-10 read machinery so
// that every computation below is deterministic.
// ---------------------------------------------------------------------------

/// `16 + 16 = 32`.
theorem nat_16_add_16 {
    Nat.16 + Nat.16 = Nat.32
} by {
    Nat.16 = Nat.1.read(Nat.6)
    read_add_read(Nat.1, Nat.6, Nat.1, Nat.6)
    Nat.1.read(Nat.6) + Nat.1.read(Nat.6) = (Nat.1 + Nat.1).read(Nat.6 + Nat.6)
    Nat.1 + Nat.1 = Nat.2
    Nat.6 + Nat.6 = Nat.12
    (Nat.1 + Nat.1).read(Nat.6 + Nat.6) = Nat.2.read(Nat.12)
    Nat.2.read(Nat.12) = Nat.2.read(Nat.10 * Nat.1 + Nat.2)
    read_read_carry(Nat.2, Nat.1, Nat.2)
    Nat.2.read(Nat.10 * Nat.1 + Nat.2) = (Nat.2 + Nat.1).read(Nat.2)
    Nat.2 + Nat.1 = Nat.3
    (Nat.2 + Nat.1).read(Nat.2) = Nat.3.read(Nat.2)
    Nat.3.read(Nat.2) = Nat.32
    Nat.16 + Nat.16 = Nat.32
}

/// `32 + 32 = 64`.
theorem nat_32_add_32 {
    Nat.32 + Nat.32 = Nat.64
} by {
    Nat.32 = Nat.3.read(Nat.2)
    read_add_read(Nat.3, Nat.2, Nat.3, Nat.2)
    Nat.3.read(Nat.2) + Nat.3.read(Nat.2) = (Nat.3 + Nat.3).read(Nat.2 + Nat.2)
    Nat.3 + Nat.3 = Nat.6
    Nat.2 + Nat.2 = Nat.4
    (Nat.3 + Nat.3).read(Nat.2 + Nat.2) = Nat.6.read(Nat.4)
    Nat.6.read(Nat.4) = Nat.64
    Nat.32 + Nat.32 = Nat.64
}

/// `64 + 64 = 128`.
theorem nat_64_add_64 {
    Nat.64 + Nat.64 = Nat.128
} by {
    Nat.64 = Nat.6.read(Nat.4)
    read_add_read(Nat.6, Nat.4, Nat.6, Nat.4)
    Nat.6.read(Nat.4) + Nat.6.read(Nat.4) = (Nat.6 + Nat.6).read(Nat.4 + Nat.4)
    Nat.6 + Nat.6 = Nat.12
    Nat.4 + Nat.4 = Nat.8
    (Nat.6 + Nat.6).read(Nat.4 + Nat.4) = Nat.12.read(Nat.8)
    Nat.12.read(Nat.8) = Nat.128
    Nat.64 + Nat.64 = Nat.128
}

/// `12 + 12 = 24`.
theorem nat_12_add_12 {
    Nat.12 + Nat.12 = Nat.24
} by {
    Nat.12 = Nat.1.read(Nat.2)
    read_add_read(Nat.1, Nat.2, Nat.1, Nat.2)
    Nat.1.read(Nat.2) + Nat.1.read(Nat.2) = (Nat.1 + Nat.1).read(Nat.2 + Nat.2)
    Nat.1 + Nat.1 = Nat.2
    Nat.2 + Nat.2 = Nat.4
    (Nat.1 + Nat.1).read(Nat.2 + Nat.2) = Nat.2.read(Nat.4)
    Nat.2.read(Nat.4) = Nat.24
    Nat.12 + Nat.12 = Nat.24
}

/// `24 + 1 = 25`.
theorem nat_24_add_1 {
    Nat.24 + Nat.1 = Nat.25
} by {
    Nat.24 = Nat.2.read(Nat.4)
    read_add_single(Nat.2, Nat.4, Nat.1)
    Nat.2.read(Nat.4) + Nat.1 = Nat.2.read(Nat.4 + Nat.1)
    Nat.4 + Nat.1 = Nat.5
    Nat.2.read(Nat.4 + Nat.1) = Nat.2.read(Nat.5)
    Nat.2.read(Nat.5) = Nat.25
    Nat.24 + Nat.1 = Nat.25
}

/// `128 + 128 = 256`.
theorem nat_128_add_128 {
    Nat.128 + Nat.128 = Nat.256
} by {
    Nat.128 = Nat.12.read(Nat.8)
    read_add_read(Nat.12, Nat.8, Nat.12, Nat.8)
    Nat.12.read(Nat.8) + Nat.12.read(Nat.8) = (Nat.12 + Nat.12).read(Nat.8 + Nat.8)
    nat_12_add_12
    Nat.12 + Nat.12 = Nat.24
    Nat.8 + Nat.8 = Nat.16
    (Nat.12 + Nat.12).read(Nat.8 + Nat.8) = Nat.24.read(Nat.16)
    Nat.24.read(Nat.16) = Nat.24.read(Nat.10 * Nat.1 + Nat.6)
    read_read_carry(Nat.24, Nat.1, Nat.6)
    Nat.24.read(Nat.10 * Nat.1 + Nat.6) = (Nat.24 + Nat.1).read(Nat.6)
    nat_24_add_1
    Nat.24 + Nat.1 = Nat.25
    (Nat.24 + Nat.1).read(Nat.6) = Nat.25.read(Nat.6)
    Nat.25.read(Nat.6) = Nat.256
    Nat.128 + Nat.128 = Nat.256
}

/// `25 + 25 = 50`.
theorem nat_25_add_25 {
    Nat.25 + Nat.25 = Nat.50
} by {
    Nat.25 = Nat.2.read(Nat.5)
    read_add_read(Nat.2, Nat.5, Nat.2, Nat.5)
    Nat.2.read(Nat.5) + Nat.2.read(Nat.5) = (Nat.2 + Nat.2).read(Nat.5 + Nat.5)
    Nat.2 + Nat.2 = Nat.4
    Nat.5 + Nat.5 = Nat.10
    (Nat.2 + Nat.2).read(Nat.5 + Nat.5) = Nat.4.read(Nat.10)
    Nat.4.read(Nat.10) = Nat.4.read(Nat.10 * Nat.1 + Nat.0)
    read_read_carry(Nat.4, Nat.1, Nat.0)
    Nat.4.read(Nat.10 * Nat.1 + Nat.0) = (Nat.4 + Nat.1).read(Nat.0)
    Nat.4 + Nat.1 = Nat.5
    (Nat.4 + Nat.1).read(Nat.0) = Nat.5.read(Nat.0)
    Nat.5.read(Nat.0) = Nat.50
    Nat.25 + Nat.25 = Nat.50
}

/// `50 + 1 = 51`.
theorem nat_50_add_1 {
    Nat.50 + Nat.1 = Nat.51
} by {
    Nat.50 = Nat.5.read(Nat.0)
    read_add_single(Nat.5, Nat.0, Nat.1)
    Nat.5.read(Nat.0) + Nat.1 = Nat.5.read(Nat.0 + Nat.1)
    Nat.0 + Nat.1 = Nat.1
    Nat.5.read(Nat.0 + Nat.1) = Nat.5.read(Nat.1)
    Nat.5.read(Nat.1) = Nat.51
    Nat.50 + Nat.1 = Nat.51
}

/// `256 + 256 = 512`.
theorem nat_256_add_256 {
    Nat.256 + Nat.256 = Nat.512
} by {
    Nat.256 = Nat.25.read(Nat.6)
    read_add_read(Nat.25, Nat.6, Nat.25, Nat.6)
    Nat.25.read(Nat.6) + Nat.25.read(Nat.6) = (Nat.25 + Nat.25).read(Nat.6 + Nat.6)
    nat_25_add_25
    Nat.25 + Nat.25 = Nat.50
    Nat.6 + Nat.6 = Nat.12
    (Nat.25 + Nat.25).read(Nat.6 + Nat.6) = Nat.50.read(Nat.12)
    Nat.50.read(Nat.12) = Nat.50.read(Nat.10 * Nat.1 + Nat.2)
    read_read_carry(Nat.50, Nat.1, Nat.2)
    Nat.50.read(Nat.10 * Nat.1 + Nat.2) = (Nat.50 + Nat.1).read(Nat.2)
    nat_50_add_1
    Nat.50 + Nat.1 = Nat.51
    (Nat.50 + Nat.1).read(Nat.2) = Nat.51.read(Nat.2)
    Nat.51.read(Nat.2) = Nat.512
    Nat.256 + Nat.256 = Nat.512
}

/// `51 + 51 = 102`.
theorem nat_51_add_51 {
    Nat.51 + Nat.51 = Nat.102
} by {
    Nat.51 = Nat.5.read(Nat.1)
    read_add_read(Nat.5, Nat.1, Nat.5, Nat.1)
    Nat.5.read(Nat.1) + Nat.5.read(Nat.1) = (Nat.5 + Nat.5).read(Nat.1 + Nat.1)
    Nat.5 + Nat.5 = Nat.10
    Nat.1 + Nat.1 = Nat.2
    (Nat.5 + Nat.5).read(Nat.1 + Nat.1) = Nat.10.read(Nat.2)
    Nat.10.read(Nat.2) = Nat.102
    Nat.51 + Nat.51 = Nat.102
}

/// `10 + 10 = 20`.
theorem nat_10_add_10 {
    Nat.10 + Nat.10 = Nat.20
} by {
    Nat.10 = Nat.1.read(Nat.0)
    read_add_read(Nat.1, Nat.0, Nat.1, Nat.0)
    Nat.1.read(Nat.0) + Nat.1.read(Nat.0) = (Nat.1 + Nat.1).read(Nat.0 + Nat.0)
    Nat.1 + Nat.1 = Nat.2
    Nat.0 + Nat.0 = Nat.0
    (Nat.1 + Nat.1).read(Nat.0 + Nat.0) = Nat.2.read(Nat.0)
    Nat.2.read(Nat.0) = Nat.20
    Nat.10 + Nat.10 = Nat.20
}

/// `102 + 102 = 204`.
theorem nat_102_add_102 {
    Nat.102 + Nat.102 = Nat.204
} by {
    Nat.102 = Nat.10.read(Nat.2)
    read_add_read(Nat.10, Nat.2, Nat.10, Nat.2)
    Nat.10.read(Nat.2) + Nat.10.read(Nat.2) = (Nat.10 + Nat.10).read(Nat.2 + Nat.2)
    nat_10_add_10
    Nat.10 + Nat.10 = Nat.20
    Nat.2 + Nat.2 = Nat.4
    (Nat.10 + Nat.10).read(Nat.2 + Nat.2) = Nat.20.read(Nat.4)
    Nat.20.read(Nat.4) = Nat.204
    Nat.102 + Nat.102 = Nat.204
}

/// `512 + 512 = 1024`.
theorem nat_512_add_512 {
    Nat.512 + Nat.512 = Nat.1024
} by {
    Nat.512 = Nat.51.read(Nat.2)
    read_add_read(Nat.51, Nat.2, Nat.51, Nat.2)
    Nat.51.read(Nat.2) + Nat.51.read(Nat.2) = (Nat.51 + Nat.51).read(Nat.2 + Nat.2)
    nat_51_add_51
    Nat.51 + Nat.51 = Nat.102
    Nat.2 + Nat.2 = Nat.4
    (Nat.51 + Nat.51).read(Nat.2 + Nat.2) = Nat.102.read(Nat.4)
    Nat.102.read(Nat.4) = Nat.1024
    Nat.512 + Nat.512 = Nat.1024
}

/// `1024 + 1024 = 2048`.
theorem nat_1024_add_1024 {
    Nat.1024 + Nat.1024 = Nat.2048
} by {
    Nat.1024 = Nat.102.read(Nat.4)
    read_add_read(Nat.102, Nat.4, Nat.102, Nat.4)
    Nat.102.read(Nat.4) + Nat.102.read(Nat.4) = (Nat.102 + Nat.102).read(Nat.4 + Nat.4)
    nat_102_add_102
    Nat.102 + Nat.102 = Nat.204
    Nat.4 + Nat.4 = Nat.8
    (Nat.102 + Nat.102).read(Nat.4 + Nat.4) = Nat.204.read(Nat.8)
    Nat.204.read(Nat.8) = Nat.2048
    Nat.1024 + Nat.1024 = Nat.2048
}

/// Doubling `16` gives `32`.
theorem nat_2_mul_16 {
    Nat.2 * Nat.16 = Nat.32
} by {
    mul_two_left(Nat.16)
    Nat.2 * Nat.16 = Nat.16 + Nat.16
    nat_16_add_16
    Nat.16 + Nat.16 = Nat.32
    Nat.2 * Nat.16 = Nat.32
}

/// Doubling `32` gives `64`.
theorem nat_2_mul_32 {
    Nat.2 * Nat.32 = Nat.64
} by {
    mul_two_left(Nat.32)
    Nat.2 * Nat.32 = Nat.32 + Nat.32
    nat_32_add_32
    Nat.32 + Nat.32 = Nat.64
    Nat.2 * Nat.32 = Nat.64
}

/// Doubling `64` gives `128`.
theorem nat_2_mul_64 {
    Nat.2 * Nat.64 = Nat.128
} by {
    mul_two_left(Nat.64)
    Nat.2 * Nat.64 = Nat.64 + Nat.64
    nat_64_add_64
    Nat.64 + Nat.64 = Nat.128
    Nat.2 * Nat.64 = Nat.128
}

/// Doubling `128` gives `256`.
theorem nat_2_mul_128 {
    Nat.2 * Nat.128 = Nat.256
} by {
    mul_two_left(Nat.128)
    Nat.2 * Nat.128 = Nat.128 + Nat.128
    nat_128_add_128
    Nat.128 + Nat.128 = Nat.256
    Nat.2 * Nat.128 = Nat.256
}

/// Doubling `256` gives `512`.
theorem nat_2_mul_256 {
    Nat.2 * Nat.256 = Nat.512
} by {
    mul_two_left(Nat.256)
    Nat.2 * Nat.256 = Nat.256 + Nat.256
    nat_256_add_256
    Nat.256 + Nat.256 = Nat.512
    Nat.2 * Nat.256 = Nat.512
}

/// Doubling `512` gives `1024`.
theorem nat_2_mul_512 {
    Nat.2 * Nat.512 = Nat.1024
} by {
    mul_two_left(Nat.512)
    Nat.2 * Nat.512 = Nat.512 + Nat.512
    nat_512_add_512
    Nat.512 + Nat.512 = Nat.1024
    Nat.2 * Nat.512 = Nat.1024
}

/// Doubling `1024` gives `2048`.
theorem nat_2_mul_1024 {
    Nat.2 * Nat.1024 = Nat.2048
} by {
    mul_two_left(Nat.1024)
    Nat.2 * Nat.1024 = Nat.1024 + Nat.1024
    nat_1024_add_1024
    Nat.1024 + Nat.1024 = Nat.2048
    Nat.2 * Nat.1024 = Nat.2048
}

/// `2^5 = 32`.
theorem nat_2_pow_5 {
    Nat.2.pow(Nat.5) = Nat.32
} by {
    exp_add(Nat.2, Nat.4, Nat.1)
    Nat.2.pow(Nat.4 + Nat.1) = Nat.2.pow(Nat.4) * Nat.2.pow(Nat.1)
    exp_one(Nat.2)
    Nat.2.pow(Nat.1) = Nat.2
    Nat.2.pow(Nat.4 + Nat.1) = Nat.2.pow(Nat.4) * Nat.2
    Nat.4 + Nat.1 = Nat.5
    Nat.2.pow(Nat.5) = Nat.2.pow(Nat.4) * Nat.2
    mul_comm(Nat.2.pow(Nat.4), Nat.2)
    Nat.2.pow(Nat.5) = Nat.2 * Nat.2.pow(Nat.4)
    two_pow_four
    Nat.2.pow(Nat.4) = Nat.16
    nat_2_mul_16
    Nat.2 * Nat.16 = Nat.32
    Nat.2.pow(Nat.5) = Nat.32
}

/// `2^11 = 2048`, obtained by repeated doubling.
theorem nat_2_pow_11 {
    Nat.2.pow(Nat.11) = Nat.2048
} by {
    two_pow_two
    Nat.2.pow(Nat.2) = Nat.4
    two_pow_three
    Nat.2.pow(Nat.3) = Nat.8
    two_pow_four
    Nat.2.pow(Nat.4) = Nat.16
    exp_add(Nat.2, Nat.4, Nat.1)
    Nat.2.pow(Nat.4 + Nat.1) = Nat.2.pow(Nat.4) * Nat.2.pow(Nat.1)
    exp_one(Nat.2)
    Nat.2.pow(Nat.1) = Nat.2
    Nat.2.pow(Nat.4 + Nat.1) = Nat.2.pow(Nat.4) * Nat.2
    Nat.4 + Nat.1 = Nat.5
    Nat.2.pow(Nat.5) = Nat.2.pow(Nat.4) * Nat.2
    mul_comm(Nat.2.pow(Nat.4), Nat.2)
    Nat.2.pow(Nat.5) = Nat.2 * Nat.2.pow(Nat.4)
    nat_2_mul_16
    Nat.2 * Nat.16 = Nat.32
    Nat.2.pow(Nat.5) = Nat.32
    exp_add(Nat.2, Nat.5, Nat.1)
    Nat.2.pow(Nat.5 + Nat.1) = Nat.2.pow(Nat.5) * Nat.2.pow(Nat.1)
    Nat.2.pow(Nat.5 + Nat.1) = Nat.2.pow(Nat.5) * Nat.2
    Nat.5 + Nat.1 = Nat.6
    Nat.2.pow(Nat.6) = Nat.2.pow(Nat.5) * Nat.2
    mul_comm(Nat.2.pow(Nat.5), Nat.2)
    Nat.2.pow(Nat.6) = Nat.2 * Nat.2.pow(Nat.5)
    nat_2_mul_32
    Nat.2 * Nat.32 = Nat.64
    Nat.2.pow(Nat.6) = Nat.64
    exp_add(Nat.2, Nat.6, Nat.1)
    Nat.2.pow(Nat.6 + Nat.1) = Nat.2.pow(Nat.6) * Nat.2.pow(Nat.1)
    Nat.2.pow(Nat.6 + Nat.1) = Nat.2.pow(Nat.6) * Nat.2
    Nat.6 + Nat.1 = Nat.7
    Nat.2.pow(Nat.7) = Nat.2.pow(Nat.6) * Nat.2
    mul_comm(Nat.2.pow(Nat.6), Nat.2)
    Nat.2.pow(Nat.7) = Nat.2 * Nat.2.pow(Nat.6)
    nat_2_mul_64
    Nat.2 * Nat.64 = Nat.128
    Nat.2.pow(Nat.7) = Nat.128
    exp_add(Nat.2, Nat.7, Nat.1)
    Nat.2.pow(Nat.7 + Nat.1) = Nat.2.pow(Nat.7) * Nat.2.pow(Nat.1)
    Nat.2.pow(Nat.7 + Nat.1) = Nat.2.pow(Nat.7) * Nat.2
    Nat.7 + Nat.1 = Nat.8
    Nat.2.pow(Nat.8) = Nat.2.pow(Nat.7) * Nat.2
    mul_comm(Nat.2.pow(Nat.7), Nat.2)
    Nat.2.pow(Nat.8) = Nat.2 * Nat.2.pow(Nat.7)
    nat_2_mul_128
    Nat.2 * Nat.128 = Nat.256
    Nat.2.pow(Nat.8) = Nat.256
    exp_add(Nat.2, Nat.8, Nat.1)
    Nat.2.pow(Nat.8 + Nat.1) = Nat.2.pow(Nat.8) * Nat.2.pow(Nat.1)
    Nat.2.pow(Nat.8 + Nat.1) = Nat.2.pow(Nat.8) * Nat.2
    Nat.8 + Nat.1 = Nat.9
    Nat.2.pow(Nat.9) = Nat.2.pow(Nat.8) * Nat.2
    mul_comm(Nat.2.pow(Nat.8), Nat.2)
    Nat.2.pow(Nat.9) = Nat.2 * Nat.2.pow(Nat.8)
    nat_2_mul_256
    Nat.2 * Nat.256 = Nat.512
    Nat.2.pow(Nat.9) = Nat.512
    exp_add(Nat.2, Nat.9, Nat.1)
    Nat.2.pow(Nat.9 + Nat.1) = Nat.2.pow(Nat.9) * Nat.2.pow(Nat.1)
    Nat.2.pow(Nat.9 + Nat.1) = Nat.2.pow(Nat.9) * Nat.2
    Nat.9 + Nat.1 = Nat.10
    Nat.2.pow(Nat.10) = Nat.2.pow(Nat.9) * Nat.2
    mul_comm(Nat.2.pow(Nat.9), Nat.2)
    Nat.2.pow(Nat.10) = Nat.2 * Nat.2.pow(Nat.9)
    nat_2_mul_512
    Nat.2 * Nat.512 = Nat.1024
    Nat.2.pow(Nat.10) = Nat.1024
    exp_add(Nat.2, Nat.10, Nat.1)
    Nat.2.pow(Nat.10 + Nat.1) = Nat.2.pow(Nat.10) * Nat.2.pow(Nat.1)
    Nat.2.pow(Nat.10 + Nat.1) = Nat.2.pow(Nat.10) * Nat.2
    Nat.10 + Nat.1 = Nat.11
    Nat.2.pow(Nat.11) = Nat.2.pow(Nat.10) * Nat.2
    mul_comm(Nat.2.pow(Nat.10), Nat.2)
    Nat.2.pow(Nat.11) = Nat.2 * Nat.2.pow(Nat.10)
    nat_2_mul_1024
    Nat.2 * Nat.1024 = Nat.2048
    Nat.2.pow(Nat.11) = Nat.2048
}

/// `30 + 1 = 31`.
theorem nat_30_add_1 {
    Nat.30 + Nat.1 = Nat.31
} by {
    Nat.30 = Nat.3.read(Nat.0)
    read_add_single(Nat.3, Nat.0, Nat.1)
    Nat.3.read(Nat.0) + Nat.1 = Nat.3.read(Nat.0 + Nat.1)
    Nat.0 + Nat.1 = Nat.1
    Nat.3.read(Nat.0 + Nat.1) = Nat.3.read(Nat.1)
    Nat.3.read(Nat.1) = Nat.31
    Nat.30 + Nat.1 = Nat.31
}

/// `31 + 1 = 32`.
theorem nat_31_add_1 {
    Nat.31 + Nat.1 = Nat.32
} by {
    Nat.31 = Nat.3.read(Nat.1)
    read_add_single(Nat.3, Nat.1, Nat.1)
    Nat.3.read(Nat.1) + Nat.1 = Nat.3.read(Nat.1 + Nat.1)
    Nat.1 + Nat.1 = Nat.2
    Nat.3.read(Nat.1 + Nat.1) = Nat.3.read(Nat.2)
    Nat.3.read(Nat.2) = Nat.32
    Nat.31 + Nat.1 = Nat.32
}

/// `2047 + 1 = 2048`.
theorem nat_2047_add_1 {
    Nat.2047 + Nat.1 = Nat.2048
} by {
    Nat.2047 = Nat.204.read(Nat.7)
    read_add_single(Nat.204, Nat.7, Nat.1)
    Nat.204.read(Nat.7) + Nat.1 = Nat.204.read(Nat.7 + Nat.1)
    Nat.7 + Nat.1 = Nat.8
    Nat.204.read(Nat.7 + Nat.1) = Nat.204.read(Nat.8)
    Nat.204.read(Nat.8) = Nat.2048
    Nat.2047 + Nat.1 = Nat.2048
}

/// `1 + 30 = 31`.
theorem nat_1_add_30 {
    Nat.1 + Nat.30 = Nat.31
} by {
    add_comm(Nat.1, Nat.30)
    Nat.1 + Nat.30 = Nat.30 + Nat.1
    nat_30_add_1
    Nat.30 + Nat.1 = Nat.31
    Nat.1 + Nat.30 = Nat.31
}

/// `1 + 2046 = 2047`.
theorem nat_1_add_2046 {
    Nat.1 + Nat.2046 = Nat.2047
} by {
    add_comm(Nat.1, Nat.2046)
    Nat.1 + Nat.2046 = Nat.2046 + Nat.1
    Nat.2046 = Nat.204.read(Nat.6)
    read_add_single(Nat.204, Nat.6, Nat.1)
    Nat.204.read(Nat.6) + Nat.1 = Nat.204.read(Nat.6 + Nat.1)
    Nat.6 + Nat.1 = Nat.7
    Nat.204.read(Nat.6 + Nat.1) = Nat.204.read(Nat.7)
    Nat.204.read(Nat.7) = Nat.2047
    Nat.2046 + Nat.1 = Nat.2047
    Nat.1 + Nat.2046 = Nat.2047
}

/// `2^5 - 1 = 31`.
theorem two_pow_5_sub_one {
    Nat.2.pow(Nat.5) - Nat.1 = Nat.31
} by {
    nat_2_pow_5
    Nat.2.pow(Nat.5) = Nat.32
    nat_31_add_1
    Nat.31 + Nat.1 = Nat.32
    add_imp_sub(Nat.31, Nat.1, Nat.32)
    Nat.32 - Nat.1 = Nat.31
    Nat.2.pow(Nat.5) - Nat.1 = Nat.31
}

/// `2^11 - 1 = 2047`.
theorem two_pow_11_sub_one {
    Nat.2.pow(Nat.11) - Nat.1 = Nat.2047
} by {
    nat_2_pow_11
    Nat.2.pow(Nat.11) = Nat.2048
    nat_2047_add_1
    Nat.2047 + Nat.1 = Nat.2048
    add_imp_sub(Nat.2047, Nat.1, Nat.2048)
    Nat.2048 - Nat.1 = Nat.2047
    Nat.2.pow(Nat.11) - Nat.1 = Nat.2047
}

/// `16 + 2 = 18`.
theorem nat_16_add_2 {
    Nat.16 + Nat.2 = Nat.18
} by {
    Nat.16 = Nat.1.read(Nat.6)
    read_add_single(Nat.1, Nat.6, Nat.2)
    Nat.1.read(Nat.6) + Nat.2 = Nat.1.read(Nat.6 + Nat.2)
    Nat.6 + Nat.2 = Nat.8
    Nat.1.read(Nat.6 + Nat.2) = Nat.1.read(Nat.8)
    Nat.1.read(Nat.8) = Nat.18
    Nat.16 + Nat.2 = Nat.18
}

/// `18 + 2 = 20`.
theorem nat_18_add_2 {
    Nat.18 + Nat.2 = Nat.20
} by {
    Nat.18 = Nat.1.read(Nat.8)
    read_add_single(Nat.1, Nat.8, Nat.2)
    Nat.1.read(Nat.8) + Nat.2 = Nat.1.read(Nat.8 + Nat.2)
    Nat.8 + Nat.2 = Nat.10
    Nat.1.read(Nat.8 + Nat.2) = Nat.1.read(Nat.10)
    Nat.1.read(Nat.10) = Nat.1.read(Nat.10 * Nat.1 + Nat.0)
    read_read_carry(Nat.1, Nat.1, Nat.0)
    Nat.1.read(Nat.10 * Nat.1 + Nat.0) = (Nat.1 + Nat.1).read(Nat.0)
    Nat.1 + Nat.1 = Nat.2
    (Nat.1 + Nat.1).read(Nat.0) = Nat.2.read(Nat.0)
    Nat.2.read(Nat.0) = Nat.20
    Nat.18 + Nat.2 = Nat.20
}

/// `8 * 23 = 184`.
theorem nat_8_mul_23 {
    Nat.8 * Nat.23 = Nat.184
} by {
    mul_comm(Nat.8, Nat.23)
    Nat.8 * Nat.23 = Nat.23 * Nat.8
    Nat.23 = Nat.2.read(Nat.3)
    read_mul_single(Nat.2, Nat.3, Nat.8)
    Nat.2.read(Nat.3) * Nat.8 = (Nat.2 * Nat.8).read(Nat.3 * Nat.8)
    nat_mul_2_8
    Nat.2 * Nat.8 = Nat.16
    nat_mul_3_8
    Nat.3 * Nat.8 = Nat.24
    (Nat.2 * Nat.8).read(Nat.3 * Nat.8) = Nat.16.read(Nat.24)
    Nat.16.read(Nat.24) = Nat.16.read(Nat.10 * Nat.2 + Nat.4)
    read_read_carry(Nat.16, Nat.2, Nat.4)
    Nat.16.read(Nat.10 * Nat.2 + Nat.4) = (Nat.16 + Nat.2).read(Nat.4)
    nat_16_add_2
    Nat.16 + Nat.2 = Nat.18
    (Nat.16 + Nat.2).read(Nat.4) = Nat.18.read(Nat.4)
    Nat.18.read(Nat.4) = Nat.184
    Nat.23 * Nat.8 = Nat.184
    Nat.8 * Nat.23 = Nat.184
}

/// `9 * 23 = 207`.
theorem nat_9_mul_23 {
    Nat.9 * Nat.23 = Nat.207
} by {
    mul_comm(Nat.9, Nat.23)
    Nat.9 * Nat.23 = Nat.23 * Nat.9
    Nat.23 = Nat.2.read(Nat.3)
    read_mul_single(Nat.2, Nat.3, Nat.9)
    Nat.2.read(Nat.3) * Nat.9 = (Nat.2 * Nat.9).read(Nat.3 * Nat.9)
    nat_mul_2_9
    Nat.2 * Nat.9 = Nat.18
    nat_mul_3_9
    Nat.3 * Nat.9 = Nat.27
    (Nat.2 * Nat.9).read(Nat.3 * Nat.9) = Nat.18.read(Nat.27)
    Nat.18.read(Nat.27) = Nat.18.read(Nat.10 * Nat.2 + Nat.7)
    read_read_carry(Nat.18, Nat.2, Nat.7)
    Nat.18.read(Nat.10 * Nat.2 + Nat.7) = (Nat.18 + Nat.2).read(Nat.7)
    nat_18_add_2
    Nat.18 + Nat.2 = Nat.20
    (Nat.18 + Nat.2).read(Nat.7) = Nat.20.read(Nat.7)
    Nat.20.read(Nat.7) = Nat.207
    Nat.23 * Nat.9 = Nat.207
    Nat.9 * Nat.23 = Nat.207
}

/// `184 + 20 = 204`.
theorem nat_184_add_20 {
    Nat.184 + Nat.20 = Nat.204
} by {
    Nat.184 = Nat.18.read(Nat.4)
    read_add_single(Nat.18, Nat.4, Nat.20)
    Nat.18.read(Nat.4) + Nat.20 = Nat.18.read(Nat.4 + Nat.20)
    add_comm(Nat.4, Nat.20)
    Nat.4 + Nat.20 = Nat.20 + Nat.4
    Nat.20 = Nat.2.read(Nat.0)
    read_add_single(Nat.2, Nat.0, Nat.4)
    Nat.2.read(Nat.0) + Nat.4 = Nat.2.read(Nat.0 + Nat.4)
    Nat.0 + Nat.4 = Nat.4
    Nat.2.read(Nat.0 + Nat.4) = Nat.2.read(Nat.4)
    Nat.2.read(Nat.4) = Nat.24
    Nat.20 + Nat.4 = Nat.24
    Nat.4 + Nat.20 = Nat.24
    Nat.18.read(Nat.4 + Nat.20) = Nat.18.read(Nat.24)
    Nat.18.read(Nat.24) = Nat.18.read(Nat.10 * Nat.2 + Nat.4)
    read_read_carry(Nat.18, Nat.2, Nat.4)
    Nat.18.read(Nat.10 * Nat.2 + Nat.4) = (Nat.18 + Nat.2).read(Nat.4)
    nat_18_add_2
    Nat.18 + Nat.2 = Nat.20
    (Nat.18 + Nat.2).read(Nat.4) = Nat.20.read(Nat.4)
    Nat.20.read(Nat.4) = Nat.204
    Nat.184 + Nat.20 = Nat.204
}

/// `23 * 89 = 2047`: the decimal multiplication `23 * 89` computed digitwise.
theorem nat_mul_23_89 {
    Nat.23 * Nat.89 = Nat.2047
} by {
    mul_comm(Nat.23, Nat.89)
    Nat.23 * Nat.89 = Nat.89 * Nat.23
    Nat.89 = Nat.8.read(Nat.9)
    read_mul_single(Nat.8, Nat.9, Nat.23)
    Nat.8.read(Nat.9) * Nat.23 = (Nat.8 * Nat.23).read(Nat.9 * Nat.23)
    nat_8_mul_23
    Nat.8 * Nat.23 = Nat.184
    nat_9_mul_23
    Nat.9 * Nat.23 = Nat.207
    (Nat.8 * Nat.23).read(Nat.9 * Nat.23) = Nat.184.read(Nat.207)
    Nat.184.read(Nat.207) = Nat.184.read(Nat.10 * Nat.20 + Nat.7)
    read_read_carry(Nat.184, Nat.20, Nat.7)
    Nat.184.read(Nat.10 * Nat.20 + Nat.7) = (Nat.184 + Nat.20).read(Nat.7)
    nat_184_add_20
    Nat.184 + Nat.20 = Nat.204
    (Nat.184 + Nat.20).read(Nat.7) = Nat.204.read(Nat.7)
    Nat.204.read(Nat.7) = Nat.2047
    Nat.89 * Nat.23 = Nat.2047
    Nat.23 * Nat.89 = Nat.2047
}

/// A positive leading digit makes a read number positive.
theorem read_pos(a: Nat, b: Nat) {
    Nat.0 < a implies Nat.0 < a.read(b)
} by {
    if Nat.0 < a {
        lt_imp_lte_suc(Nat.0, a)
        Nat.1 <= a
        lte_mul_both(Nat.10, Nat.1, a)
        Nat.10 * Nat.1 <= Nat.10 * a
        Nat.10 * Nat.1 = Nat.10
        Nat.10 <= Nat.10 * a
        lte_add_left(Nat.10 * a, Nat.0, b)
        Nat.10 * a + Nat.0 <= Nat.10 * a + b
        add_zero_right(Nat.10 * a)
        Nat.10 * a <= Nat.10 * a + b
        lte_trans(Nat.10, Nat.10 * a, Nat.10 * a + b)
        Nat.10 <= Nat.10 * a + b
        lt_zero_one
        Nat.0 < Nat.1
        Nat.1 + Nat.9 = Nat.10
        exists(c: Nat) { Nat.1 + c = Nat.10 }
        Nat.1 <= Nat.10
        lt_ne(Nat.1, Nat.10, Nat.9)
        Nat.1 + Nat.9 = Nat.10
        Nat.9 != Nat.0
        Nat.1 != Nat.10
        Nat.1 < Nat.10
        lt_trans(Nat.0, Nat.1, Nat.10)
        Nat.0 < Nat.10
        lt_and_lte(Nat.0, Nat.10, Nat.10 * a + b)
        Nat.0 < Nat.10 * a + b
        a.read(b) = Nat.10 * a + b
        Nat.0 < a.read(b)
    }
}

// ---------------------------------------------------------------------------
// Primality of the small numbers 2, 3 and 31.
// ---------------------------------------------------------------------------

/// `2 != 1`.
theorem two_ne_one {
    Nat.2 != Nat.1
} by {
    lt_ne(Nat.1, Nat.2, Nat.1)
    Nat.1 + Nat.1 = Nat.2
    Nat.1 != Nat.0
    Nat.1 != Nat.2
    Nat.2 != Nat.1
}

/// `3 != 1`.
theorem three_ne_one {
    Nat.3 != Nat.1
} by {
    lt_ne(Nat.1, Nat.3, Nat.2)
    Nat.1 + Nat.2 = Nat.3
    Nat.2 != Nat.0
    Nat.1 != Nat.3
    Nat.3 != Nat.1
}

/// `1 < 3`.
theorem nat_1_lt_3 {
    Nat.1 < Nat.3
} by {
    Nat.1 + Nat.2 = Nat.3
    exists(c: Nat) { Nat.1 + c = Nat.3 }
    Nat.1 <= Nat.3
    three_ne_one
    Nat.3 != Nat.1
    Nat.1 < Nat.3
}

/// Two does not divide three.
theorem not_two_divides_three {
    not Nat.2.divides(Nat.3)
} by {
    if Nat.2.divides(Nat.3) {
        Nat.2 * Nat.1 = Nat.2
        exists(c: Nat) { Nat.2 * c = Nat.2 }
        Nat.2.divides(Nat.2)
        divides_suc_pair_imp_one(Nat.2, Nat.2)
        Nat.2 = Nat.1
        two_ne_one
        false
    }
}

/// No number strictly between one and three divides three.
theorem three_no_proper_divisor(k: Nat) {
    Nat.1 < k and k < Nat.3 implies not k.divides(Nat.3)
} by {
    if Nat.1 < k and k < Nat.3 {
        lt_suc_right(k, Nat.2)
        k = Nat.2 or k < Nat.2
        if k < Nat.2 {
            lt_imp_lte_suc(Nat.1, k)
            Nat.2 <= k
            lte_and_lt(Nat.2, k, Nat.2)
            Nat.2 < Nat.2
            lt_not_ref(Nat.2)
            false
        } else {
            k = Nat.2
            not_two_divides_three
            not Nat.2.divides(Nat.3)
            not k.divides(Nat.3)
        }
    }
}

/// Three is prime.
theorem three_is_prime {
    Nat.3.is_prime
} by {
    nat_1_lt_3
    Nat.1 < Nat.3
    forall(k: Nat) {
        three_no_proper_divisor(k)
    }
    no_proper_divisor_imp_prime(Nat.3)
}

/// Two is prime.
theorem two_is_prime {
    Nat.2.is_prime
} by {
    lt_one_two
    Nat.1 < Nat.2
    forall(k: Nat) {
        if Nat.1 < k and k < Nat.2 {
            lt_imp_lte_suc(Nat.1, k)
            Nat.2 <= k
            lte_and_lt(Nat.2, k, Nat.2)
            Nat.2 < Nat.2
            lt_not_ref(Nat.2)
            false
        }
    }
    no_proper_divisor_imp_prime(Nat.2)
}

/// `3 * 10 = 30`.
theorem nat_mul_3_10 {
    Nat.3 * Nat.10 = Nat.30
} by {
    Nat.10 = Nat.1.read(Nat.0)
    read_mul_single(Nat.1, Nat.0, Nat.3)
    Nat.1.read(Nat.0) * Nat.3 = (Nat.1 * Nat.3).read(Nat.0 * Nat.3)
    nat_mul_1_3
    Nat.1 * Nat.3 = Nat.3
    mul_zero_right(Nat.3)
    Nat.0 * Nat.3 = Nat.0
    (Nat.1 * Nat.3).read(Nat.0 * Nat.3) = Nat.3.read(Nat.0)
    Nat.3.read(Nat.0) = Nat.30
    Nat.10 * Nat.3 = Nat.30
    mul_comm(Nat.3, Nat.10)
    Nat.3 * Nat.10 = Nat.10 * Nat.3
    Nat.3 * Nat.10 = Nat.30
}

/// `11 * 3 = 33`.
theorem nat_mul_11_3 {
    Nat.11 * Nat.3 = Nat.33
} by {
    Nat.11 = Nat.1.read(Nat.1)
    read_mul_single(Nat.1, Nat.1, Nat.3)
    Nat.1.read(Nat.1) * Nat.3 = (Nat.1 * Nat.3).read(Nat.1 * Nat.3)
    nat_mul_1_3
    Nat.1 * Nat.3 = Nat.3
    (Nat.1 * Nat.3).read(Nat.1 * Nat.3) = Nat.3.read(Nat.3)
    Nat.3.read(Nat.3) = Nat.33
    Nat.11 * Nat.3 = Nat.33
}

/// `13 * 3 = 39`.
theorem nat_mul_13_3 {
    Nat.13 * Nat.3 = Nat.39
} by {
    Nat.13 = Nat.1.read(Nat.3)
    read_mul_single(Nat.1, Nat.3, Nat.3)
    Nat.1.read(Nat.3) * Nat.3 = (Nat.1 * Nat.3).read(Nat.3 * Nat.3)
    nat_mul_1_3
    Nat.1 * Nat.3 = Nat.3
    nat_mul_3_3
    Nat.3 * Nat.3 = Nat.9
    (Nat.1 * Nat.3).read(Nat.3 * Nat.3) = Nat.3.read(Nat.9)
    Nat.3.read(Nat.9) = Nat.39
    Nat.13 * Nat.3 = Nat.39
}

/// `2 * 15 = 30`.
theorem nat_2_mul_15 {
    Nat.2 * Nat.15 = Nat.30
} by {
    mul_two_left(Nat.15)
    Nat.2 * Nat.15 = Nat.15 + Nat.15
    Nat.15 = Nat.1.read(Nat.5)
    read_add_read(Nat.1, Nat.5, Nat.1, Nat.5)
    Nat.1.read(Nat.5) + Nat.1.read(Nat.5) = (Nat.1 + Nat.1).read(Nat.5 + Nat.5)
    Nat.1 + Nat.1 = Nat.2
    Nat.5 + Nat.5 = Nat.10
    (Nat.1 + Nat.1).read(Nat.5 + Nat.5) = Nat.2.read(Nat.10)
    Nat.2.read(Nat.10) = Nat.2.read(Nat.10 * Nat.1 + Nat.0)
    read_read_carry(Nat.2, Nat.1, Nat.0)
    Nat.2.read(Nat.10 * Nat.1 + Nat.0) = (Nat.2 + Nat.1).read(Nat.0)
    Nat.2 + Nat.1 = Nat.3
    (Nat.2 + Nat.1).read(Nat.0) = Nat.3.read(Nat.0)
    Nat.3.read(Nat.0) = Nat.30
    Nat.15 + Nat.15 = Nat.30
    Nat.2 * Nat.15 = Nat.30
}

/// `4 + 31 = 35`.
theorem nat_4_add_31 {
    Nat.4 + Nat.31 = Nat.35
} by {
    Nat.31 = Nat.3.read(Nat.1)
    read_add_single(Nat.3, Nat.1, Nat.4)
    Nat.3.read(Nat.1) + Nat.4 = Nat.3.read(Nat.1 + Nat.4)
    Nat.1 + Nat.4 = Nat.5
    Nat.3.read(Nat.1 + Nat.4) = Nat.3.read(Nat.5)
    Nat.3.read(Nat.5) = Nat.35
    Nat.4 + Nat.31 = Nat.35
}

/// `2 + 31 = 33`.
theorem nat_2_add_31 {
    Nat.2 + Nat.31 = Nat.33
} by {
    Nat.31 = Nat.3.read(Nat.1)
    read_add_single(Nat.3, Nat.1, Nat.2)
    Nat.3.read(Nat.1) + Nat.2 = Nat.3.read(Nat.1 + Nat.2)
    Nat.1 + Nat.2 = Nat.3
    Nat.3.read(Nat.1 + Nat.2) = Nat.3.read(Nat.3)
    Nat.3.read(Nat.3) = Nat.33
    Nat.2 + Nat.31 = Nat.33
}

/// `8 + 31 = 39`.
theorem nat_8_add_31 {
    Nat.8 + Nat.31 = Nat.39
} by {
    Nat.31 = Nat.3.read(Nat.1)
    read_add_single(Nat.3, Nat.1, Nat.8)
    Nat.3.read(Nat.1) + Nat.8 = Nat.3.read(Nat.1 + Nat.8)
    Nat.1 + Nat.8 = Nat.9
    Nat.3.read(Nat.1 + Nat.8) = Nat.3.read(Nat.9)
    Nat.3.read(Nat.9) = Nat.39
    Nat.8 + Nat.31 = Nat.39
}

/// `35 - 31 = 4`.
theorem nat_35_sub_31 {
    Nat.35 - Nat.31 = Nat.4
} by {
    nat_4_add_31
    Nat.4 + Nat.31 = Nat.35
    add_imp_sub(Nat.4, Nat.31, Nat.35)
    Nat.35 - Nat.31 = Nat.4
}

/// `33 - 31 = 2`.
theorem nat_33_sub_31 {
    Nat.33 - Nat.31 = Nat.2
} by {
    nat_2_add_31
    Nat.2 + Nat.31 = Nat.33
    add_imp_sub(Nat.2, Nat.31, Nat.33)
    Nat.33 - Nat.31 = Nat.2
}

/// `39 - 31 = 8`.
theorem nat_39_sub_31 {
    Nat.39 - Nat.31 = Nat.8
} by {
    nat_8_add_31
    Nat.8 + Nat.31 = Nat.39
    add_imp_sub(Nat.8, Nat.31, Nat.39)
    Nat.39 - Nat.31 = Nat.8
}

/// Two does not divide thirty-one: `2` divides `30` but not `31 = 30 + 1`.
theorem not_two_divides_31 {
    not Nat.2.divides(Nat.31)
} by {
    if Nat.2.divides(Nat.31) {
        nat_2_mul_15
        Nat.2 * Nat.15 = Nat.30
        exists(c: Nat) { Nat.2 * c = Nat.30 }
        Nat.2.divides(Nat.30)
        nat_30_add_1
        Nat.30 + Nat.1 = Nat.31
        divides_suc_pair_imp_one(Nat.2, Nat.30)
        Nat.2 = Nat.1
        two_ne_one
        false
    }
}

/// Three does not divide thirty-one: `3` divides `30` but not `31`.
theorem not_three_divides_31 {
    not Nat.3.divides(Nat.31)
} by {
    if Nat.3.divides(Nat.31) {
        nat_mul_3_10
        Nat.3 * Nat.10 = Nat.30
        exists(c: Nat) { Nat.3 * c = Nat.30 }
        Nat.3.divides(Nat.30)
        nat_30_add_1
        Nat.30 + Nat.1 = Nat.31
        divides_suc_pair_imp_one(Nat.3, Nat.30)
        Nat.3 = Nat.1
        three_ne_one
        false
    }
}

/// Five does not divide thirty-one.
theorem not_five_divides_31 {
    not Nat.5.divides(Nat.31)
} by {
    if Nat.5.divides(Nat.31) {
        nat_mul_5_6
        Nat.5 * Nat.6 = Nat.30
        exists(c: Nat) { Nat.5 * c = Nat.30 }
        Nat.5.divides(Nat.30)
        nat_30_add_1
        Nat.30 + Nat.1 = Nat.31
        divides_suc_pair_imp_one(Nat.5, Nat.30)
        Nat.5 = Nat.1
        lt_ne(Nat.1, Nat.5, Nat.4)
        Nat.1 + Nat.4 = Nat.5
        Nat.4 != Nat.0
        Nat.1 != Nat.5
        Nat.5 != Nat.1
        false
    }
}

/// Seven does not divide thirty-one: `7` divides `35` and `31`, hence `4`.
theorem not_seven_divides_31 {
    not Nat.7.divides(Nat.31)
} by {
    if Nat.7.divides(Nat.31) {
        nat_mul_7_5
        Nat.7 * Nat.5 = Nat.35
        exists(c: Nat) { Nat.7 * c = Nat.35 }
        Nat.7.divides(Nat.35)
        divides_sub(Nat.35, Nat.31, Nat.7)
        Nat.7.divides(Nat.35) and Nat.7.divides(Nat.31) implies Nat.7.divides(Nat.35 - Nat.31)
        Nat.7.divides(Nat.35 - Nat.31)
        nat_35_sub_31
        Nat.35 - Nat.31 = Nat.4
        Nat.7.divides(Nat.4)
        not_divides_of_lt(Nat.7, Nat.4)
        Nat.0 < Nat.4 and Nat.4 < Nat.7 implies not Nat.7.divides(Nat.4)
        Nat.0 < Nat.4
        Nat.4 < Nat.7
        false
    }
}

/// `2 < 11`.
theorem nat_2_lt_11 {
    Nat.2 < Nat.11
} by {
    Nat.2 + Nat.9 = Nat.11
    exists(c: Nat) { Nat.2 + c = Nat.11 }
    Nat.2 <= Nat.11
    lt_ne(Nat.2, Nat.11, Nat.9)
    Nat.2 + Nat.9 = Nat.11
    Nat.9 != Nat.0
    Nat.2 != Nat.11
    Nat.2 < Nat.11
}

/// `5 + 3 = 8`.
theorem nat_5_add_3 {
    Nat.5 + Nat.3 = Nat.8
} by {
    Nat.3 = Nat.1 + Nat.2
    add_assoc(Nat.5, Nat.1, Nat.2)
    Nat.5 + (Nat.1 + Nat.2) = (Nat.5 + Nat.1) + Nat.2
    Nat.5 + Nat.1 = Nat.6
    (Nat.5 + Nat.1) + Nat.2 = Nat.6 + Nat.2
    Nat.6 + Nat.2 = Nat.8
    Nat.5 + Nat.3 = Nat.8
}

/// `10 + 3 = 13`.
theorem nat_10_add_3 {
    Nat.10 + Nat.3 = Nat.13
} by {
    Nat.10 = Nat.1.read(Nat.0)
    read_add_single(Nat.1, Nat.0, Nat.3)
    Nat.1.read(Nat.0) + Nat.3 = Nat.1.read(Nat.0 + Nat.3)
    Nat.0 + Nat.3 = Nat.3
    Nat.1.read(Nat.0 + Nat.3) = Nat.1.read(Nat.3)
    Nat.1.read(Nat.3) = Nat.13
    Nat.10 + Nat.3 = Nat.13
}

/// `5 + 8 = 13`, via `5 + 8 = 5 + (5 + 3) = 10 + 3`.
theorem nat_5_add_8 {
    Nat.5 + Nat.8 = Nat.13
} by {
    nat_5_add_3
    Nat.5 + Nat.3 = Nat.8
    Nat.5 + Nat.8 = Nat.5 + (Nat.5 + Nat.3)
    add_assoc(Nat.5, Nat.5, Nat.3)
    Nat.5 + (Nat.5 + Nat.3) = (Nat.5 + Nat.5) + Nat.3
    Nat.5 + Nat.5 = Nat.10
    (Nat.5 + Nat.5) + Nat.3 = Nat.10 + Nat.3
    nat_10_add_3
    Nat.10 + Nat.3 = Nat.13
    Nat.5 + Nat.8 = Nat.13
}

/// `8 + 5 = 13`.
theorem nat_8_add_5 {
    Nat.8 + Nat.5 = Nat.13
} by {
    add_comm(Nat.8, Nat.5)
    Nat.8 + Nat.5 = Nat.5 + Nat.8
    nat_5_add_8
    Nat.5 + Nat.8 = Nat.13
    Nat.8 + Nat.5 = Nat.13
}

/// `8 < 13`.
theorem nat_8_lt_13 {
    Nat.8 < Nat.13
} by {
    nat_8_add_5
    Nat.8 + Nat.5 = Nat.13
    exists(c: Nat) { Nat.8 + c = Nat.13 }
    Nat.8 <= Nat.13
    lt_ne(Nat.8, Nat.13, Nat.5)
    nat_8_add_5
    Nat.8 + Nat.5 = Nat.13
    Nat.5 != Nat.0
    Nat.8 != Nat.13
    Nat.8 < Nat.13
}

/// Eleven does not divide thirty-one: `11` divides `33` and `31`, hence `2`.
theorem not_eleven_divides_31 {
    not Nat.11.divides(Nat.31)
} by {
    if Nat.11.divides(Nat.31) {
        nat_mul_11_3
        Nat.11 * Nat.3 = Nat.33
        exists(c: Nat) { Nat.11 * c = Nat.33 }
        Nat.11.divides(Nat.33)
        divides_sub(Nat.33, Nat.31, Nat.11)
        Nat.11.divides(Nat.33) and Nat.11.divides(Nat.31) implies Nat.11.divides(Nat.33 - Nat.31)
        Nat.11.divides(Nat.33 - Nat.31)
        nat_33_sub_31
        Nat.33 - Nat.31 = Nat.2
        Nat.11.divides(Nat.2)
        not_divides_of_lt(Nat.11, Nat.2)
        Nat.0 < Nat.2 and Nat.2 < Nat.11 implies not Nat.11.divides(Nat.2)
        lt_zero_two
        Nat.0 < Nat.2
        nat_2_lt_11
        Nat.2 < Nat.11
        false
    }
}

/// Thirteen does not divide thirty-one: `13` divides `39` and `31`, hence `8`.
theorem not_thirteen_divides_31 {
    not Nat.13.divides(Nat.31)
} by {
    if Nat.13.divides(Nat.31) {
        nat_mul_13_3
        Nat.13 * Nat.3 = Nat.39
        exists(c: Nat) { Nat.13 * c = Nat.39 }
        Nat.13.divides(Nat.39)
        divides_sub(Nat.39, Nat.31, Nat.13)
        Nat.13.divides(Nat.39) and Nat.13.divides(Nat.31) implies Nat.13.divides(Nat.39 - Nat.31)
        Nat.13.divides(Nat.39 - Nat.31)
        nat_39_sub_31
        Nat.39 - Nat.31 = Nat.8
        Nat.13.divides(Nat.8)
        not_divides_of_lt(Nat.13, Nat.8)
        Nat.0 < Nat.8 and Nat.8 < Nat.13 implies not Nat.13.divides(Nat.8)
        Nat.0 < Nat.8
        nat_8_lt_13
        Nat.8 < Nat.13
        false
    }
}

/// Four does not divide thirty-one: a divisor of `4` dividing `31` would make
/// `2` divide `31`.
theorem not_four_divides_31 {
    not Nat.4.divides(Nat.31)
} by {
    if Nat.4.divides(Nat.31) {
        nat_mul_2_2
        Nat.2 * Nat.2 = Nat.4
        exists(c: Nat) { Nat.2 * c = Nat.4 }
        Nat.2.divides(Nat.4)
        divides_trans(Nat.2, Nat.4, Nat.31)
        Nat.2.divides(Nat.4) and Nat.4.divides(Nat.31) implies Nat.2.divides(Nat.31)
        Nat.2.divides(Nat.31)
        not_two_divides_31
        false
    }
}

/// No nontrivial factorization `31 = b * c` exists: one of the factors is
/// at most `5` (otherwise `b * c >= 36 > 31`).  Each candidate factor
/// `2, 3, 4, 5` is ruled out individually.
theorem thirty_one_composite_contradiction(b: Nat, c: Nat) {
    Nat.1 < b and Nat.1 < c implies not (Nat.31 = b * c)
} by {
    if Nat.1 < b and Nat.1 < c {
        if Nat.31 = b * c {
            Nat.31 = b * c
            exists(x: Nat) { b * x = Nat.31 }
            b.divides(Nat.31)
            if b <= Nat.5 {
                lt_suc(Nat.5)
                Nat.5 < Nat.6
                lte_and_lt(b, Nat.5, Nat.6)
                b < Nat.6
                lt_suc_right(b, Nat.5)
                if b = Nat.5 {
                    Nat.5.divides(Nat.31)
                    not_five_divides_31
                    false
                } else {
                    b < Nat.5
                    lt_suc_right(b, Nat.4)
                    if b = Nat.4 {
                        Nat.4.divides(Nat.31)
                        not_four_divides_31
                        false
                    } else {
                        b < Nat.4
                        lt_suc_right(b, Nat.3)
                        if b = Nat.3 {
                            Nat.3.divides(Nat.31)
                            not_three_divides_31
                            false
                        } else {
                            b < Nat.3
                            lt_suc_right(b, Nat.2)
                            if b = Nat.2 {
                                Nat.2.divides(Nat.31)
                                not_two_divides_31
                                false
                            } else {
                                b < Nat.2
                                lt_imp_lte_suc(Nat.1, b)
                                Nat.2 <= b
                                lte_and_lt(Nat.2, b, Nat.2)
                                Nat.2 < Nat.2
                                lt_not_ref(Nat.2)
                                false
                            }
                        }
                    }
                }
            } else {
            not (b <= Nat.5)
            lt_or_lte(Nat.5, b)
            Nat.5 < b or b <= Nat.5
            Nat.5 < b
            lt_imp_lte_suc(Nat.5, b)
            Nat.6 <= b
            if c <= Nat.5 {
                Nat.31 = b * c
                exists(x: Nat) { c * x = Nat.31 }
                c.divides(Nat.31)
                lt_suc(Nat.5)
                Nat.5 < Nat.6
                lte_and_lt(c, Nat.5, Nat.6)
                c < Nat.6
                lt_suc_right(c, Nat.5)
                if c = Nat.5 {
                    Nat.5.divides(Nat.31)
                    not_five_divides_31
                    false
                } else {
                    c < Nat.5
                    lt_suc_right(c, Nat.4)
                    if c = Nat.4 {
                        Nat.4.divides(Nat.31)
                        not_four_divides_31
                        false
                    } else {
                        c < Nat.4
                        lt_suc_right(c, Nat.3)
                        if c = Nat.3 {
                            Nat.3.divides(Nat.31)
                            not_three_divides_31
                            false
                        } else {
                            c < Nat.3
                            lt_suc_right(c, Nat.2)
                            if c = Nat.2 {
                                Nat.2.divides(Nat.31)
                                not_two_divides_31
                                false
                            } else {
                                c < Nat.2
                                lt_imp_lte_suc(Nat.1, c)
                                Nat.2 <= c
                                lte_and_lt(Nat.2, c, Nat.2)
                                Nat.2 < Nat.2
                                lt_not_ref(Nat.2)
                                false
                            }
                        }
                    }
                }
            } else {
                not (c <= Nat.5)
                lt_or_lte(Nat.5, c)
                Nat.5 < c or c <= Nat.5
                Nat.5 < c
                lt_imp_lte_suc(Nat.5, c)
                Nat.6 <= c
                lte_mul_both(Nat.6, Nat.6, c)
                Nat.6 <= c implies Nat.6 * Nat.6 <= Nat.6 * c
                Nat.6 * Nat.6 <= Nat.6 * c
                nat_mul_6_6
                Nat.6 * Nat.6 = Nat.36
                Nat.36 <= Nat.6 * c
                lte_mul_both(c, Nat.6, b)
                Nat.6 <= b implies c * Nat.6 <= c * b
                c * Nat.6 <= c * b
                mul_comm(c, Nat.6)
                c * Nat.6 = Nat.6 * c
                mul_comm(c, b)
                c * b = b * c
                Nat.6 * c <= b * c
                lte_trans(Nat.36, Nat.6 * c, b * c)
                Nat.36 <= b * c
                Nat.31 = b * c
                Nat.36 <= Nat.31
                Nat.31 + Nat.5 = Nat.36
                exists(d: Nat) { Nat.31 + d = Nat.36 }
                Nat.31 <= Nat.36
                lt_ne(Nat.31, Nat.36, Nat.5)
                Nat.31 + Nat.5 = Nat.36
                Nat.5 != Nat.0
                Nat.31 != Nat.36
                Nat.31 < Nat.36
                lte_imp_not_lt(Nat.36, Nat.31)
                not (Nat.31 < Nat.36)
                false
            }
        }
        }
        not (Nat.31 = b * c)
    }
}

/// `30 != 0`, since `30 = 3.read(0)`.
theorem nat_30_ne_zero {
    Nat.30 != Nat.0
} by {
    Nat.30 = Nat.3.read(Nat.0)
    read_pos(Nat.3, Nat.0)
    Nat.0 < Nat.3
    Nat.0 < Nat.30
    Nat.30 != Nat.0
}

/// `1 != 31`.
theorem nat_1_neq_31 {
    Nat.1 != Nat.31
} by {
    lt_ne(Nat.1, Nat.31, Nat.30)
    nat_1_add_30
    Nat.1 + Nat.30 = Nat.31
    nat_30_ne_zero
    Nat.30 != Nat.0
    Nat.1 != Nat.31
}

/// `1 < 31`.
theorem nat_1_lt_31 {
    Nat.1 < Nat.31
} by {
    nat_1_add_30
    Nat.1 + Nat.30 = Nat.31
    exists(c: Nat) { Nat.1 + c = Nat.31 }
    Nat.1 <= Nat.31
    nat_1_neq_31
    Nat.1 != Nat.31
    Nat.1 < Nat.31
}

/// Thirty-one is prime.
theorem thirty_one_is_prime {
    Nat.31.is_prime
} by {
    nat_1_lt_31
    Nat.1 < Nat.31
    if Nat.31.is_composite {
        Nat.31.is_composite = exists(b: Nat, c: Nat) {
            Nat.1 < b and Nat.1 < c and Nat.31 = b * c
        }
        let (b: Nat, c: Nat) satisfy { Nat.1 < b and Nat.1 < c and Nat.31 = b * c }
        thirty_one_composite_contradiction(b, c)
        Nat.1 < b and Nat.1 < c implies not (Nat.31 = b * c)
        not (Nat.31 = b * c)
        Nat.31 = b * c
        false
    }
    not Nat.31.is_composite
    Nat.31.is_prime = Nat.1 < Nat.31 and not Nat.31.is_composite
    Nat.31.is_prime
}

// ---------------------------------------------------------------------------
// (b) 2^p - 1 is prime for p = 2, 3, 5: the Mersenne primes 3, 7 and 31.
// ---------------------------------------------------------------------------

/// `2^2 - 1 = 3`.
theorem two_pow_two_sub_one_eq_three {
    Nat.2.pow(Nat.2) - Nat.1 = Nat.3
} by {
    two_pow_two
    Nat.2.pow(Nat.2) = Nat.4
    Nat.3 + Nat.1 = Nat.4
    add_imp_sub(Nat.3, Nat.1, Nat.4)
    Nat.4 - Nat.1 = Nat.3
    Nat.2.pow(Nat.2) - Nat.1 = Nat.3
}

/// `2^3 - 1 = 7`.
theorem two_pow_three_sub_one_eq_seven {
    Nat.2.pow(Nat.3) - Nat.1 = Nat.7
} by {
    two_pow_three
    Nat.2.pow(Nat.3) = Nat.8
    Nat.7 + Nat.1 = Nat.8
    add_imp_sub(Nat.7, Nat.1, Nat.8)
    Nat.8 - Nat.1 = Nat.7
    Nat.2.pow(Nat.3) - Nat.1 = Nat.7
}

/// `2^5 - 1 = 31`.
theorem two_pow_five_sub_one_eq_thirty_one {
    Nat.2.pow(Nat.5) - Nat.1 = Nat.31
} by {
    two_pow_5_sub_one
    Nat.2.pow(Nat.5) - Nat.1 = Nat.31
}

/// `2^2 - 1 = 3` is prime.
theorem two_pow_two_sub_one_is_prime {
    (Nat.2.pow(Nat.2) - Nat.1).is_prime
} by {
    two_pow_two_sub_one_eq_three
    Nat.2.pow(Nat.2) - Nat.1 = Nat.3
    three_is_prime
    Nat.3.is_prime
    (Nat.2.pow(Nat.2) - Nat.1).is_prime
}

/// `2^3 - 1 = 7` is prime.
theorem two_pow_three_sub_one_is_prime {
    (Nat.2.pow(Nat.3) - Nat.1).is_prime
} by {
    two_pow_three_sub_one_eq_seven
    Nat.2.pow(Nat.3) - Nat.1 = Nat.7
    seven_is_prime
    Nat.7.is_prime
    (Nat.2.pow(Nat.3) - Nat.1).is_prime
}

/// `2^5 - 1 = 31` is prime.
theorem two_pow_five_sub_one_is_prime {
    (Nat.2.pow(Nat.5) - Nat.1).is_prime
} by {
    two_pow_five_sub_one_eq_thirty_one
    Nat.2.pow(Nat.5) - Nat.1 = Nat.31
    thirty_one_is_prime
    Nat.31.is_prime
    (Nat.2.pow(Nat.5) - Nat.1).is_prime
}

// ---------------------------------------------------------------------------
// (c) 2^11 - 1 = 2047 = 23 * 89 is the first composite Mersenne number.
// ---------------------------------------------------------------------------

/// `1 < 23`.
theorem nat_1_lt_23 {
    Nat.1 < Nat.23
} by {
    Nat.1 + Nat.22 = Nat.23
    exists(c: Nat) { Nat.1 + c = Nat.23 }
    Nat.1 <= Nat.23
    lt_ne(Nat.1, Nat.23, Nat.22)
    Nat.1 + Nat.22 = Nat.23
    Nat.22 = Nat.2.read(Nat.2)
    read_pos(Nat.2, Nat.2)
    lt_zero_two
    Nat.0 < Nat.2
    Nat.0 < Nat.22
    Nat.22 != Nat.0
    Nat.1 != Nat.23
    Nat.1 < Nat.23
}

/// `1 < 89`.
theorem nat_1_lt_89 {
    Nat.1 < Nat.89
} by {
    Nat.1 + Nat.88 = Nat.89
    exists(c: Nat) { Nat.1 + c = Nat.89 }
    Nat.1 <= Nat.89
    lt_ne(Nat.1, Nat.89, Nat.88)
    Nat.1 + Nat.88 = Nat.89
    Nat.88 = Nat.8.read(Nat.8)
    read_pos(Nat.8, Nat.8)
    Nat.0 < Nat.8
    Nat.0 < Nat.88
    Nat.88 != Nat.0
    Nat.1 != Nat.89
    Nat.1 < Nat.89
}

/// `2047` is composite, witnessed by `2047 = 23 * 89`.
theorem nat_2047_is_composite {
    Nat.2047.is_composite
} by {
    Nat.2047.is_composite = exists(b: Nat, c: Nat) {
        Nat.1 < b and Nat.1 < c and Nat.2047 = b * c
    }
    nat_1_lt_23
    Nat.1 < Nat.23
    nat_1_lt_89
    Nat.1 < Nat.89
    nat_mul_23_89
    Nat.23 * Nat.89 = Nat.2047
    Nat.2047 = Nat.23 * Nat.89
    Nat.1 < Nat.23 and Nat.1 < Nat.89 and Nat.2047 = Nat.23 * Nat.89
    exists(b: Nat, c: Nat) { Nat.1 < b and Nat.1 < c and Nat.2047 = b * c }
    Nat.2047.is_composite
}

/// `2047` is not prime.
theorem nat_2047_not_prime {
    not Nat.2047.is_prime
} by {
    if Nat.2047.is_prime {
        Nat.2047.is_prime = Nat.1 < Nat.2047 and not Nat.2047.is_composite
        Nat.1 < Nat.2047 and not Nat.2047.is_composite
        not Nat.2047.is_composite
        nat_2047_is_composite
        Nat.2047.is_composite
        false
    }
}

/// `2^11 - 1 = 2047` is composite.
theorem two_pow_11_sub_one_is_composite {
    (Nat.2.pow(Nat.11) - Nat.1).is_composite
} by {
    two_pow_11_sub_one
    Nat.2.pow(Nat.11) - Nat.1 = Nat.2047
    nat_2047_is_composite
    Nat.2047.is_composite
    (Nat.2.pow(Nat.11) - Nat.1).is_composite
}

/// `2^11 - 1` is not prime: the first composite Mersenne number.
theorem two_pow_11_sub_one_not_prime {
    not (Nat.2.pow(Nat.11) - Nat.1).is_prime
} by {
    if (Nat.2.pow(Nat.11) - Nat.1).is_prime {
        two_pow_11_sub_one
        Nat.2.pow(Nat.11) - Nat.1 = Nat.2047
        Nat.2047.is_prime
        nat_2047_not_prime
        false
    }
}

// ---------------------------------------------------------------------------
// (a) If 2^p - 1 is prime then p is prime.
// ---------------------------------------------------------------------------

/// The finite geometric-series identity in the naturals, for any base `p > 1`:
/// `(p - 1) * (p^0 + ... + p^(k-1)) + 1 = p^k`.
theorem nat_pow_sum_geometric_gen(p: Nat, k: Nat) {
    Nat.1 < p implies
        (p - Nat.1) * partial(p.pow, k) + Nat.1 = p.pow(k)
} by {
    if Nat.1 < p {
        let f: Nat -> Bool = function(x: Nat) {
            (p - Nat.1) * partial(p.pow, x) + Nat.1 = p.pow(x)
        }
        partial(p.pow, Nat.0) = sum(map(Nat.0.range, p.pow))
        map(Nat.0.range, p.pow) = map(List.nil[Nat], p.pow)
        map(List.nil[Nat], p.pow) = List.nil[Nat]
        sum(List.nil[Nat]) = Nat.0
        partial(p.pow, Nat.0) = Nat.0
        (p - Nat.1) * Nat.0 = Nat.0
        Nat.0 + Nat.1 = Nat.1
        exp_zero(p)
        p.pow(Nat.0) = Nat.1
        f(Nat.0)
        forall(x: Nat) {
            if f(x) {
                partial_split_last(p.pow, x)
                partial(p.pow, x.suc) = partial(p.pow, x) + p.pow(x)
                (p - Nat.1) * (partial(p.pow, x) + p.pow(x)) =
                    (p - Nat.1) * partial(p.pow, x) +
                        (p - Nat.1) * p.pow(x)
                (p - Nat.1) * partial(p.pow, x) + Nat.1 = p.pow(x)
                (p - Nat.1) * partial(p.pow, x) +
                    (p - Nat.1) * p.pow(x) + Nat.1 =
                    (p - Nat.1) * partial(p.pow, x) +
                        ((p - Nat.1) * p.pow(x) + Nat.1)
                add_assoc((p - Nat.1) * partial(p.pow, x),
                    (p - Nat.1) * p.pow(x), Nat.1)
                (p - Nat.1) * p.pow(x) + Nat.1 =
                    Nat.1 + (p - Nat.1) * p.pow(x)
                add_comm((p - Nat.1) * p.pow(x), Nat.1)
                (p - Nat.1) * partial(p.pow, x) +
                    (Nat.1 + (p - Nat.1) * p.pow(x)) =
                    (p - Nat.1) * partial(p.pow, x) + Nat.1 +
                        (p - Nat.1) * p.pow(x)
                add_assoc((p - Nat.1) * partial(p.pow, x), Nat.1,
                    (p - Nat.1) * p.pow(x))
                (p - Nat.1) * partial(p.pow, x) + Nat.1 +
                    (p - Nat.1) * p.pow(x) =
                    p.pow(x) + (p - Nat.1) * p.pow(x)
                (p - Nat.1) * p.pow(x) = p.pow(x) * (p - Nat.1)
                p.pow(x) + (p - Nat.1) * p.pow(x) =
                    p.pow(x) + p.pow(x) * (p - Nat.1)
                p.pow(x) + p.pow(x) * (p - Nat.1) =
                    p.pow(x) * (Nat.1 + (p - Nat.1))
                lt_imp_lte_suc(Nat.1, p)
                Nat.1.suc <= p
                Nat.2 <= p
                lte_trans(Nat.1, Nat.2, p)
                Nat.1 <= p
                sub_one_add_one(p)
                (p - Nat.1) + Nat.1 = p
                Nat.1 + (p - Nat.1) = (p - Nat.1) + Nat.1
                add_comm(Nat.1, p - Nat.1)
                Nat.1 + (p - Nat.1) = p
                p.pow(x) * p = p.pow(x + Nat.1)
                exp_add(p, x, Nat.1)
                exp_one(p)
                p.pow(x + Nat.1) = p.pow(x.suc)
                x + Nat.1 = x.suc
                (p - Nat.1) * partial(p.pow, x.suc) + Nat.1 =
                    p.pow(x.suc)
                f(x.suc)
            }
        }
        f(Nat.0) and forall(x: Nat) { f(x) implies f(x.suc) }
        Nat.induction(f)
        f(k)
        (p - Nat.1) * partial(p.pow, k) + Nat.1 = p.pow(k)
    }
}

/// `p^k - 1 = (p - 1) * (1 + p + ... + p^(k-1))` for `p > 1`.
theorem pow_sub_one_factor(p: Nat, k: Nat) {
    Nat.1 < p implies p.pow(k) - Nat.1 = (p - Nat.1) * partial(p.pow, k)
} by {
    if Nat.1 < p {
        nat_pow_sum_geometric_gen(p, k)
        (p - Nat.1) * partial(p.pow, k) + Nat.1 = p.pow(k)
        add_imp_sub((p - Nat.1) * partial(p.pow, k), Nat.1, p.pow(k))
        p.pow(k) - Nat.1 = (p - Nat.1) * partial(p.pow, k)
    }
}

/// `p - 1` divides `p^k - 1` for `p > 1`.
theorem pow_sub_one_divides(p: Nat, k: Nat) {
    Nat.1 < p implies (p - Nat.1).divides(p.pow(k) - Nat.1)
} by {
    if Nat.1 < p {
        pow_sub_one_factor(p, k)
        p.pow(k) - Nat.1 = (p - Nat.1) * partial(p.pow, k)
        (p - Nat.1) * partial(p.pow, k) = p.pow(k) - Nat.1
        exists(c: Nat) { (p - Nat.1) * c = p.pow(k) - Nat.1 }
        (p - Nat.1).divides(p.pow(k) - Nat.1)
    }
}

/// If `d != 0` then `2^d > 1`.
theorem two_pow_suc_gt_one(d: Nat) {
    d != Nat.0 implies Nat.1 < Nat.2.pow(d)
} by {
    if d != Nat.0 {
        pos_of_ne_zero(d)
        Nat.0 < d
        two_divides_two_pow(d)
        Nat.2.divides(Nat.2.pow(d))
        let c: Nat satisfy { Nat.2 * c = Nat.2.pow(d) }
        Nat.2.pow(d) = Nat.2 * c
        exp_ne_zero(Nat.2, d)
        Nat.2.pow(d) != Nat.0
        if c = Nat.0 {
            Nat.2 * c = Nat.0
            Nat.2.pow(d) = Nat.0
            false
        }
        c != Nat.0
        lte_mul(Nat.2, c)
        Nat.2 <= Nat.2 * c
        Nat.2 <= Nat.2.pow(d)
        lt_one_two
        Nat.1 < Nat.2
        lt_and_lte(Nat.1, Nat.2, Nat.2.pow(d))
        Nat.1 < Nat.2.pow(d)
    }
}

/// `2^a < 2^p` whenever `a < p`.
theorem two_pow_lt_exp(a: Nat, p: Nat) {
    a < p implies Nat.2.pow(a) < Nat.2.pow(p)
} by {
    if a < p {
        lt_imp_lte_suc(a, p)
        a.suc <= p
        let c: Nat satisfy { a.suc + c = p }
        add_one_right(a)
        a.suc = a + Nat.1
        (a + Nat.1) + c = p
        add_assoc(a, Nat.1, c)
        a + (Nat.1 + c) = (a + Nat.1) + c
        a + (Nat.1 + c) = p
        add_one_left(c)
        Nat.1 + c = c.suc
        a + c.suc = p
        exists(d: Nat) { a + d = p and d != Nat.0 }
        let d: Nat satisfy { a + d = p and d != Nat.0 }
        exp_add(Nat.2, a, d)
        Nat.2.pow(a + d) = Nat.2.pow(a) * Nat.2.pow(d)
        a + d = p
        Nat.2.pow(p) = Nat.2.pow(a) * Nat.2.pow(d)
        two_pow_suc_gt_one(d)
        Nat.1 < Nat.2.pow(d)
        exp_ne_zero(Nat.2, a)
        Nat.2.pow(a) != Nat.0
        lt_mul_both(Nat.2.pow(a), Nat.1, Nat.2.pow(d))
        Nat.2.pow(a) != Nat.0 and Nat.1 < Nat.2.pow(d) implies Nat.2.pow(a) * Nat.1 < Nat.2.pow(a) * Nat.2.pow(d)
        Nat.2.pow(a) * Nat.1 < Nat.2.pow(a) * Nat.2.pow(d)
        mul_one_right(Nat.2.pow(a))
        Nat.2.pow(a) * Nat.1 = Nat.2.pow(a)
        Nat.2.pow(a) < Nat.2.pow(a) * Nat.2.pow(d)
        Nat.2.pow(a) < Nat.2.pow(p)
    }
}

/// `2^a > 1` whenever `a >= 1`.
theorem two_pow_gt_one(a: Nat) {
    Nat.1 <= a implies Nat.1 < Nat.2.pow(a)
} by {
    if Nat.1 <= a {
        if a = Nat.0 {
            Nat.1 <= Nat.0
            only_zero_lte_zero(Nat.1)
            Nat.1 = Nat.0
            lt_one_two
            Nat.1 < Nat.2
            false
        }
        a != Nat.0
        two_pow_suc_gt_one(a)
        Nat.1 < Nat.2.pow(a)
    }
}

/// If `p = a * b` with `1 <= a`, then `2^a - 1` divides `2^p - 1`.
theorem mersenne_factor_divisibility(a: Nat, b: Nat) {
    Nat.1 <= a implies (Nat.2.pow(a) - Nat.1).divides(Nat.2.pow(a * b) - Nat.1)
} by {
    if Nat.1 <= a {
        two_pow_gt_one(a)
        Nat.1 < Nat.2.pow(a)
        pow_sub_one_divides(Nat.2.pow(a), b)
        Nat.1 < Nat.2.pow(a) implies (Nat.2.pow(a) - Nat.1).divides(Nat.2.pow(a).pow(b) - Nat.1)
        (Nat.2.pow(a) - Nat.1).divides(Nat.2.pow(a).pow(b) - Nat.1)
        exp_mul(Nat.2, a, b)
        Nat.2.pow(a * b) = Nat.2.pow(a).pow(b)
        (Nat.2.pow(a) - Nat.1).divides(Nat.2.pow(a * b) - Nat.1)
    }
}

/// If `2^p - 1` is prime then `p` is prime: a composite exponent `p = a b`
/// would give the proper divisor `2^a - 1` of `2^p - 1`.
theorem mersenne_prime_imp_prime(p: Nat) {
    (Nat.2.pow(p) - Nat.1).is_prime implies p.is_prime
} by {
    if (Nat.2.pow(p) - Nat.1).is_prime {
        Nat.1 < Nat.2.pow(p) - Nat.1
        if p = Nat.0 {
            exp_zero(Nat.2)
            Nat.2.pow(Nat.0) = Nat.1
            Nat.2.pow(p) - Nat.1 = Nat.1 - Nat.1
            sub_self(Nat.1)
            Nat.1 - Nat.1 = Nat.0
            Nat.2.pow(p) - Nat.1 = Nat.0
            Nat.1 < Nat.0
            not_lt_zero(Nat.1)
            false
        }
        if p = Nat.1 {
            exp_one(Nat.2)
            Nat.2.pow(Nat.1) = Nat.2
            suc_sub_one(Nat.1)
            Nat.2 - Nat.1 = Nat.1
            Nat.2.pow(p) - Nat.1 = Nat.1
            Nat.1 < Nat.1
            lt_not_ref(Nat.1)
            false
        }
        p != Nat.0
        p != Nat.1
        zero_or_suc(p)
        let r: Nat satisfy { r.suc = p }
        if r = Nat.0 {
            r.suc = Nat.1
            p = Nat.1
            false
        }
        r != Nat.0
        pos_of_ne_zero(r)
        Nat.0 < r
        lt_imp_lte_suc(Nat.0, r)
        Nat.1 <= r
        lt_suc(r)
        r < r.suc
        r < p
        lte_and_lt(Nat.1, r, p)
        Nat.1 < p
        if p.is_composite {
            p.is_composite = exists(b: Nat, c: Nat) {
                Nat.1 < b and Nat.1 < c and p = b * c
            }
            let (b: Nat, c: Nat) satisfy { Nat.1 < b and Nat.1 < c and p = b * c }
            lt_imp_lte_suc(Nat.1, b)
            Nat.1.suc <= b
            Nat.2 <= b
            lte_trans(Nat.1, Nat.2, b)
            Nat.1 <= b
            mersenne_factor_divisibility(b, c)
            Nat.1 <= b implies (Nat.2.pow(b) - Nat.1).divides(Nat.2.pow(b * c) - Nat.1)
            (Nat.2.pow(b) - Nat.1).divides(Nat.2.pow(b * c) - Nat.1)
            p = b * c
            (Nat.2.pow(b) - Nat.1).divides(Nat.2.pow(p) - Nat.1)
            two_pow_sub_one_gt_one(b)
            Nat.1 < b implies Nat.1 < Nat.2.pow(b) - Nat.1
            Nat.1 < Nat.2.pow(b) - Nat.1
            divisor_lt(b, c, p)
            b != Nat.0 and Nat.1 < c and b * c = p implies b < p
            b != Nat.0
            b < p
            two_pow_lt_exp(b, p)
            b < p implies Nat.2.pow(b) < Nat.2.pow(p)
            Nat.2.pow(b) < Nat.2.pow(p)
            prime_divisor_is_one_or_self(Nat.2.pow(p) - Nat.1, Nat.2.pow(b) - Nat.1)
            (Nat.2.pow(p) - Nat.1).is_prime and (Nat.2.pow(b) - Nat.1).divides(Nat.2.pow(p) - Nat.1) implies Nat.2.pow(b) - Nat.1 = Nat.1 or Nat.2.pow(b) - Nat.1 = Nat.2.pow(p) - Nat.1
            Nat.2.pow(b) - Nat.1 = Nat.1 or Nat.2.pow(b) - Nat.1 = Nat.2.pow(p) - Nat.1
            if Nat.2.pow(b) - Nat.1 = Nat.1 {
                Nat.1 < Nat.2.pow(b) - Nat.1
                Nat.1 < Nat.1
                lt_not_ref(Nat.1)
                false
            }
            if Nat.2.pow(b) - Nat.1 = Nat.2.pow(p) - Nat.1 {
                one_le_two_pow(b)
                Nat.1 <= Nat.2.pow(b)
                add_sub(Nat.2.pow(b), Nat.1)
                Nat.2.pow(b) - Nat.1 + Nat.1 = Nat.2.pow(b)
                one_le_two_pow(p)
                Nat.1 <= Nat.2.pow(p)
                add_sub(Nat.2.pow(p), Nat.1)
                Nat.2.pow(p) - Nat.1 + Nat.1 = Nat.2.pow(p)
                Nat.2.pow(b) - Nat.1 + Nat.1 = Nat.2.pow(p) - Nat.1 + Nat.1
                Nat.2.pow(b) = Nat.2.pow(p)
                Nat.2.pow(b) < Nat.2.pow(p)
                Nat.2.pow(b) < Nat.2.pow(b)
                lt_not_ref(Nat.2.pow(b))
                false
            }
            Nat.2.pow(b) - Nat.1 = Nat.1 or Nat.2.pow(b) - Nat.1 = Nat.2.pow(p) - Nat.1
            false
        }
        not p.is_composite
        Nat.1 < p
        p.is_prime
    }
}

// ---------------------------------------------------------------------------
// (d) Euclid's construction: a Mersenne prime yields a perfect number.
// ---------------------------------------------------------------------------

/// `2^k * 2 = 2^(k+1)`.
theorem two_pow_mul_two_eq_suc(k: Nat) {
    Nat.2.pow(k) * Nat.2 = Nat.2.pow(k + Nat.1)
} by {
    exp_add(Nat.2, k, Nat.1)
    Nat.2.pow(k + Nat.1) = Nat.2.pow(k) * Nat.2.pow(Nat.1)
    exp_one(Nat.2)
    Nat.2.pow(Nat.1) = Nat.2
    Nat.2.pow(k + Nat.1) = Nat.2.pow(k) * Nat.2
    Nat.2.pow(k) * Nat.2 = Nat.2.pow(k + Nat.1)
}

/// A divisor of `2^k` divides `2^(k+1)`.
theorem divides_pow_two_step(d: Nat, k: Nat) {
    d.divides(Nat.2.pow(k)) implies d.divides(Nat.2.pow(k) * Nat.2)
} by {
    if d.divides(Nat.2.pow(k)) {
        divides_mul(Nat.2.pow(k), Nat.2, d)
        d.divides(Nat.2.pow(k) * Nat.2)
    }
}

/// `2^k` is coprime with `2^(k+1) - 1`: any common divisor divides both
/// `2^(k+1)` and `2^(k+1) - 1`, hence is one.
theorem two_pow_coprime_two_pow_sub_one(k: Nat) {
    Nat.2.pow(k).coprime(Nat.2.pow(k + Nat.1) - Nat.1)
} by {
    let d: Nat = Nat.2.pow(k).gcd(Nat.2.pow(k + Nat.1) - Nat.1)
    gcd_divides_left(Nat.2.pow(k), Nat.2.pow(k + Nat.1) - Nat.1)
    d.divides(Nat.2.pow(k))
    gcd_divides_right(Nat.2.pow(k), Nat.2.pow(k + Nat.1) - Nat.1)
    d.divides(Nat.2.pow(k + Nat.1) - Nat.1)
    divides_pow_two_step(d, k)
    d.divides(Nat.2.pow(k) * Nat.2)
    two_pow_mul_two_eq_suc(k)
    Nat.2.pow(k) * Nat.2 = Nat.2.pow(k + Nat.1)
    d.divides(Nat.2.pow(k + Nat.1))
    one_le_two_pow(k + Nat.1)
    Nat.1 <= Nat.2.pow(k + Nat.1)
    add_sub(Nat.2.pow(k + Nat.1), Nat.1)
    Nat.2.pow(k + Nat.1) - Nat.1 + Nat.1 = Nat.2.pow(k + Nat.1)
    divides_suc_pair_imp_one(d, Nat.2.pow(k + Nat.1) - Nat.1)
    d = Nat.1
    Nat.2.pow(k).gcd(Nat.2.pow(k + Nat.1) - Nat.1) = Nat.1
    Nat.2.pow(k).coprime(Nat.2.pow(k + Nat.1) - Nat.1)
}

/// `2^(p-1)` is coprime with `2^p - 1` for `1 <= p`.
theorem two_pow_sub_one_coprime_mersenne(p: Nat) {
    Nat.1 <= p implies Nat.2.pow(p - Nat.1).coprime(Nat.2.pow(p) - Nat.1)
} by {
    if Nat.1 <= p {
        two_pow_coprime_two_pow_sub_one(p - Nat.1)
        Nat.2.pow(p - Nat.1).coprime(Nat.2.pow(p - Nat.1 + Nat.1) - Nat.1)
        add_sub(p, Nat.1)
        p - Nat.1 + Nat.1 = p
        Nat.2.pow(p - Nat.1 + Nat.1) - Nat.1 = Nat.2.pow(p) - Nat.1
        Nat.2.pow(p - Nat.1).coprime(Nat.2.pow(p) - Nat.1)
    }
}

/// Euclid's construction: if `2^p - 1` is prime then `2^(p-1)(2^p - 1)` is
/// perfect.  Since `2^(p-1)` and `2^p - 1` are coprime, the divisor sum is
/// `sigma(2^(p-1)) sigma(2^p - 1) = (2^p - 1) 2^p = 2 * 2^(p-1)(2^p - 1)`.
theorem euclid_construction(p: Nat) {
    (Nat.2.pow(p) - Nat.1).is_prime implies
        is_perfect(Nat.2.pow(p - Nat.1) * (Nat.2.pow(p) - Nat.1))
} by {
    if (Nat.2.pow(p) - Nat.1).is_prime {
        mersenne_prime_imp_prime(p)
        p.is_prime
        Nat.1 < p
        lt_imp_lte_suc(Nat.1, p)
        Nat.2 <= p
        lte_trans(Nat.1, Nat.2, p)
        Nat.1 <= p
        Nat.1 < Nat.2.pow(p) - Nat.1
        lt_zero_one
        Nat.0 < Nat.1
        lt_trans(Nat.0, Nat.1, Nat.2.pow(p) - Nat.1)
        Nat.0 < Nat.2.pow(p) - Nat.1
        two_pow_positive(p - Nat.1)
        Nat.0 < Nat.2.pow(p - Nat.1)
        two_pow_sub_one_coprime_mersenne(p)
        Nat.1 <= p implies Nat.2.pow(p - Nat.1).coprime(Nat.2.pow(p) - Nat.1)
        Nat.2.pow(p - Nat.1).coprime(Nat.2.pow(p) - Nat.1)
        nat_sigma_mul_coprime(Nat.2.pow(p - Nat.1), Nat.2.pow(p) - Nat.1)
        Nat.0 < Nat.2.pow(p - Nat.1) and Nat.0 < Nat.2.pow(p) - Nat.1 and Nat.2.pow(p - Nat.1).coprime(Nat.2.pow(p) - Nat.1) implies nat_sigma(Nat.2.pow(p - Nat.1) * (Nat.2.pow(p) - Nat.1)) = nat_sigma(Nat.2.pow(p - Nat.1)) * nat_sigma(Nat.2.pow(p) - Nat.1)
        nat_sigma(Nat.2.pow(p - Nat.1) * (Nat.2.pow(p) - Nat.1)) = nat_sigma(Nat.2.pow(p - Nat.1)) * nat_sigma(Nat.2.pow(p) - Nat.1)
        nat_sigma_prime(Nat.2.pow(p) - Nat.1)
        (Nat.2.pow(p) - Nat.1).is_prime implies nat_sigma(Nat.2.pow(p) - Nat.1) = Nat.2.pow(p) - Nat.1 + Nat.1
        nat_sigma(Nat.2.pow(p) - Nat.1) = Nat.2.pow(p) - Nat.1 + Nat.1
        one_le_two_pow(p)
        Nat.1 <= Nat.2.pow(p)
        add_sub(Nat.2.pow(p), Nat.1)
        Nat.2.pow(p) - Nat.1 + Nat.1 = Nat.2.pow(p)
        nat_sigma(Nat.2.pow(p) - Nat.1) = Nat.2.pow(p)
        two_is_prime
        Nat.2.is_prime
        nat_sigma_prime_pow_mult(Nat.2, p - Nat.1)
        (Nat.2 - Nat.1) * nat_sigma(Nat.2.pow(p - Nat.1)) + Nat.1 = Nat.2.pow(p - Nat.1 + Nat.1)
        suc_sub_one(Nat.1)
        Nat.2 - Nat.1 = Nat.1
        Nat.1 * nat_sigma(Nat.2.pow(p - Nat.1)) + Nat.1 = Nat.2.pow(p - Nat.1 + Nat.1)
        mul_one_left(nat_sigma(Nat.2.pow(p - Nat.1)))
        nat_sigma(Nat.2.pow(p - Nat.1)) + Nat.1 = Nat.2.pow(p - Nat.1 + Nat.1)
        add_sub(p, Nat.1)
        p - Nat.1 + Nat.1 = p
        Nat.2.pow(p - Nat.1 + Nat.1) = Nat.2.pow(p)
        nat_sigma(Nat.2.pow(p - Nat.1)) + Nat.1 = Nat.2.pow(p)
        add_imp_sub(nat_sigma(Nat.2.pow(p - Nat.1)), Nat.1, Nat.2.pow(p))
        Nat.2.pow(p) - Nat.1 = nat_sigma(Nat.2.pow(p - Nat.1))
        nat_sigma(Nat.2.pow(p - Nat.1)) = Nat.2.pow(p) - Nat.1
        nat_sigma(Nat.2.pow(p - Nat.1) * (Nat.2.pow(p) - Nat.1)) = (Nat.2.pow(p) - Nat.1) * Nat.2.pow(p)
        two_pow_mul_two_eq_suc(p - Nat.1)
        Nat.2.pow(p - Nat.1) * Nat.2 = Nat.2.pow(p - Nat.1 + Nat.1)
        Nat.2.pow(p - Nat.1 + Nat.1) = Nat.2.pow(p)
        Nat.2.pow(p - Nat.1) * Nat.2 = Nat.2.pow(p)
        mul_comm(Nat.2.pow(p - Nat.1), Nat.2)
        Nat.2 * Nat.2.pow(p - Nat.1) = Nat.2.pow(p)
        mul_assoc(Nat.2, Nat.2.pow(p - Nat.1), Nat.2.pow(p) - Nat.1)
        Nat.2 * (Nat.2.pow(p - Nat.1) * (Nat.2.pow(p) - Nat.1)) = (Nat.2 * Nat.2.pow(p - Nat.1)) * (Nat.2.pow(p) - Nat.1)
        Nat.2 * (Nat.2.pow(p - Nat.1) * (Nat.2.pow(p) - Nat.1)) = Nat.2.pow(p) * (Nat.2.pow(p) - Nat.1)
        mul_comm(Nat.2.pow(p), Nat.2.pow(p) - Nat.1)
        Nat.2.pow(p) * (Nat.2.pow(p) - Nat.1) = (Nat.2.pow(p) - Nat.1) * Nat.2.pow(p)
        Nat.2 * (Nat.2.pow(p - Nat.1) * (Nat.2.pow(p) - Nat.1)) = (Nat.2.pow(p) - Nat.1) * Nat.2.pow(p)
        nat_sigma(Nat.2.pow(p - Nat.1) * (Nat.2.pow(p) - Nat.1)) = Nat.2 * (Nat.2.pow(p - Nat.1) * (Nat.2.pow(p) - Nat.1))
        is_perfect(Nat.2.pow(p - Nat.1) * (Nat.2.pow(p) - Nat.1)) = (nat_sigma(Nat.2.pow(p - Nat.1) * (Nat.2.pow(p) - Nat.1)) = Nat.2 * (Nat.2.pow(p - Nat.1) * (Nat.2.pow(p) - Nat.1)))
        is_perfect(Nat.2.pow(p - Nat.1) * (Nat.2.pow(p) - Nat.1))
    }
}

// ---------------------------------------------------------------------------
// (e) The Euclid-Euler theorem.
// ---------------------------------------------------------------------------
//
// A natural number `n` is perfect exactly when it has the form
// `2^(p-1)(2^p - 1)` with `2^p - 1` prime (and then `p` is prime by (a)).
// The reverse direction is Euclid's construction (d).  The forward
// direction is Euler's theorem: if `n` is even and perfect, write
// `n = 2^(k-1) m` with `m` odd (`k >= 2`); multiplicativity of `sigma` on
// the coprime factors `2^(k-1)` and `m` gives
// `(2^k - 1) sigma(m) = 2^k m`.  Since `2^k - 1` is coprime to `2^k`, it
// divides `m`, say `m = (2^k - 1) r`.  Substituting and cancelling shows
// `sigma(m) = 2^k r`; as `r` is a proper divisor of `m` with
// `sigma(m) >= m + r`, equality forces `r = 1`, so `m = 2^k - 1` and
// `sigma(m) = m + 1`, which means `m` is prime.  This direction is left as
// future work (the multiplicativity of `sigma` is
// `nat_sigma_mul_coprime`).
//
// theorem euclid_euler_iff(n: Nat) {
//     is_perfect(n) and Nat.2.divides(n) implies
//         exists(p: Nat) {
//             (Nat.2.pow(p) - Nat.1).is_prime and
//             n = Nat.2.pow(p - Nat.1) * (Nat.2.pow(p) - Nat.1)
//         }
// }
