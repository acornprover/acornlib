// Classical identities for the Fibonacci numbers.
//
// The Fibonacci sequence is the local copy `seq_fib` of
// `number_theory/sequence_identities.ac` (the library's `fib` in
// `combinatorics/generating_functions.ac` is private to the combinatorics
// package), with `F_0 = 0`, `F_1 = 1`, and the defining recurrence
// `F_{n+2} = F_{n+1} + F_n` (`seq_fib_suc_suc`).
//
// This file proves:
//
//   (a) Cassini's identity:  F_{n+1}F_{n-1} - F_n^2 = (-1)^n.
//       The sign `(-1)^n` cannot live in the naturals, so the identity is
//       stated in the equivalent addition-free split by parity of n:
//         n odd,  n = 2k + 1:  F_{2k+1}^2 = F_{2k+2}F_{2k} + 1
//         n even, n = 2k + 2:  F_{2k+3}F_{2k+1} = F_{2k+2}^2 + 1
//       Both follow by induction on k from the pair of core statements
//         F_{2k+1}^2 = F_{2k}F_{2k+2} + 1          (seq_fib_cassini_square)
//         F_{2k+1}F_{2k+3} = F_{2k+2}^2 + 1        (seq_fib_cassini_product)
//       where each step flips one statement into the other.
//
//   (b) The addition formula: F_{m+n} = F_mF_{n+1} + F_{m-1}F_n.  Since the
//       index m - 1 is unavailable at m = 0, the proved form is the shifted
//       statement F_{m+n+1} = F_{m+1}F_{n+1} + F_mF_n
//       (seq_fib_add), which is the classical formula with m replaced by
//       m + 1.
//
//   (c) The square-difference variant: F_{n+1}^2 - F_nF_{n+2} = (-1)^n,
//       again split by parity of n; it is exactly the same pair of core
//       statements as (a) (seq_fib_square_diff_even / _odd).
//
//   (d) The gcd identity gcd(F_m, F_n) = F_{gcd(m,n)}: stated below but not
//       proved (it needs the Euclidean descent on Fibonacci numbers).
//
//   (e) The parity pattern: F_n is even iff 3 | n: stated below and verified
//       on the small cases n = 0..6 (F_0, F_3, F_6 even; F_1, F_2, F_4, F_5
//       odd), matching 3 | n exactly for those n.

from nat import Nat, add_comm, add_assoc, add_suc_right, add_suc_left, add_one_right,
    add_zero_right, add_zero_left, mul_comm, mul_assoc, distrib_left, distrib_right,
    mul_one_right, mul_one_left, mul_zero_right, mul_zero_left, mul_two_left,
    mul_suc_right, one_plus_one, suc_ne, divides_zero, divides_self, two_divides_suc_iff,
    div_imp_mod, small_mod, lt_suc, lt_imp_lt_suc
from number_theory.sequence_identities import seq_fib, seq_fib_zero, seq_fib_one, seq_fib_two,
    seq_fib_suc_suc
from number_theory.congruence import mod_add_mul

numerals Nat

// ---------------------------------------------------------------------------
// Cassini's identity (a) and the square-difference variant (c).
//
// Core statements (both parities together), for every k:
//
//   A(k): F_{2k+1}^2 = F_{2k}F_{2k+2} + 1
//   B(k): F_{2k+1}F_{2k+3} = F_{2k+2}^2 + 1
//
// The flip A(k) -> B(k) is one application of the recurrence
// F_{2k+3} = F_{2k+2} + F_{2k+1}; the flip B(k) -> A(k+1) is the same
// recurrence applied to F_{2k+4} = F_{2k+3} + F_{2k+2}.  Hence A(0) starts an
// induction that proves A(k) and B(k) for every k.
// ---------------------------------------------------------------------------

/// The flip from `F_{2k+1}^2 = F_{2k}F_{2k+2} + 1` to
/// `F_{2k+1}F_{2k+3} = F_{2k+2}^2 + 1`.
theorem seq_fib_cassini_flip_a(k: Nat) {
    seq_fib(Nat.2 * k + Nat.1) * seq_fib(Nat.2 * k + Nat.1) =
        seq_fib(Nat.2 * k) * seq_fib(Nat.2 * k + Nat.2) + Nat.1
    implies
    seq_fib(Nat.2 * k + Nat.1) * seq_fib(Nat.2 * k + Nat.3) =
        seq_fib(Nat.2 * k + Nat.2) * seq_fib(Nat.2 * k + Nat.2) + Nat.1
} by {
    if seq_fib(Nat.2 * k + Nat.1) * seq_fib(Nat.2 * k + Nat.1) =
        seq_fib(Nat.2 * k) * seq_fib(Nat.2 * k + Nat.2) + Nat.1 {
        seq_fib_suc_suc(Nat.2 * k + Nat.1)
        seq_fib(Nat.2 * k + Nat.3) = seq_fib(Nat.2 * k + Nat.2) + seq_fib(Nat.2 * k + Nat.1)
        seq_fib(Nat.2 * k + Nat.1) * seq_fib(Nat.2 * k + Nat.3) =
            seq_fib(Nat.2 * k + Nat.1) * (seq_fib(Nat.2 * k + Nat.2) + seq_fib(Nat.2 * k + Nat.1))
        distrib_left(seq_fib(Nat.2 * k + Nat.1), seq_fib(Nat.2 * k + Nat.2), seq_fib(Nat.2 * k + Nat.1))
        seq_fib(Nat.2 * k + Nat.1) * (seq_fib(Nat.2 * k + Nat.2) + seq_fib(Nat.2 * k + Nat.1)) =
            seq_fib(Nat.2 * k + Nat.1) * seq_fib(Nat.2 * k + Nat.2) +
                seq_fib(Nat.2 * k + Nat.1) * seq_fib(Nat.2 * k + Nat.1)
        seq_fib(Nat.2 * k + Nat.1) * seq_fib(Nat.2 * k + Nat.2) +
            seq_fib(Nat.2 * k + Nat.1) * seq_fib(Nat.2 * k + Nat.1) =
            seq_fib(Nat.2 * k + Nat.1) * seq_fib(Nat.2 * k + Nat.2) +
                (seq_fib(Nat.2 * k) * seq_fib(Nat.2 * k + Nat.2) + Nat.1)
        seq_fib(Nat.2 * k + Nat.1) * seq_fib(Nat.2 * k + Nat.2) +
            (seq_fib(Nat.2 * k) * seq_fib(Nat.2 * k + Nat.2) + Nat.1) =
            (seq_fib(Nat.2 * k + Nat.1) + seq_fib(Nat.2 * k)) * seq_fib(Nat.2 * k + Nat.2) + Nat.1
        seq_fib_suc_suc(Nat.2 * k)
        seq_fib(Nat.2 * k + Nat.2) = seq_fib(Nat.2 * k + Nat.1) + seq_fib(Nat.2 * k)
        (seq_fib(Nat.2 * k + Nat.1) + seq_fib(Nat.2 * k)) * seq_fib(Nat.2 * k + Nat.2) + Nat.1 =
            seq_fib(Nat.2 * k + Nat.2) * seq_fib(Nat.2 * k + Nat.2) + Nat.1
        seq_fib(Nat.2 * k + Nat.1) * seq_fib(Nat.2 * k + Nat.3) =
            seq_fib(Nat.2 * k + Nat.2) * seq_fib(Nat.2 * k + Nat.2) + Nat.1
    }
}

/// The flip from `F_{2k+1}F_{2k+3} = F_{2k+2}^2 + 1` to
/// `F_{2k+3}^2 = F_{2k+2}F_{2k+4} + 1` (the statement A at k + 1).
theorem seq_fib_cassini_flip_b(k: Nat) {
    seq_fib(Nat.2 * k + Nat.1) * seq_fib(Nat.2 * k + Nat.3) =
        seq_fib(Nat.2 * k + Nat.2) * seq_fib(Nat.2 * k + Nat.2) + Nat.1
    implies
    seq_fib(Nat.2 * k.suc + Nat.1) * seq_fib(Nat.2 * k.suc + Nat.1) =
        seq_fib(Nat.2 * k.suc) * seq_fib(Nat.2 * k.suc + Nat.2) + Nat.1
} by {
    if seq_fib(Nat.2 * k + Nat.1) * seq_fib(Nat.2 * k + Nat.3) =
        seq_fib(Nat.2 * k + Nat.2) * seq_fib(Nat.2 * k + Nat.2) + Nat.1 {
        seq_fib_suc_suc(Nat.2 * k + Nat.2)
        seq_fib(Nat.2 * k + Nat.4) = seq_fib(Nat.2 * k + Nat.3) + seq_fib(Nat.2 * k + Nat.2)
        seq_fib(Nat.2 * k + Nat.2) * seq_fib(Nat.2 * k + Nat.4) + Nat.1 =
            seq_fib(Nat.2 * k + Nat.2) * (seq_fib(Nat.2 * k + Nat.3) + seq_fib(Nat.2 * k + Nat.2)) + Nat.1
        distrib_left(seq_fib(Nat.2 * k + Nat.2), seq_fib(Nat.2 * k + Nat.3), seq_fib(Nat.2 * k + Nat.2))
        seq_fib(Nat.2 * k + Nat.2) * (seq_fib(Nat.2 * k + Nat.3) + seq_fib(Nat.2 * k + Nat.2)) =
            seq_fib(Nat.2 * k + Nat.2) * seq_fib(Nat.2 * k + Nat.3) +
                seq_fib(Nat.2 * k + Nat.2) * seq_fib(Nat.2 * k + Nat.2)
        seq_fib(Nat.2 * k + Nat.2) * seq_fib(Nat.2 * k + Nat.4) + Nat.1 =
            seq_fib(Nat.2 * k + Nat.2) * seq_fib(Nat.2 * k + Nat.3) +
                (seq_fib(Nat.2 * k + Nat.2) * seq_fib(Nat.2 * k + Nat.2) + Nat.1)
        seq_fib(Nat.2 * k + Nat.2) * seq_fib(Nat.2 * k + Nat.3) +
            (seq_fib(Nat.2 * k + Nat.2) * seq_fib(Nat.2 * k + Nat.2) + Nat.1) =
            seq_fib(Nat.2 * k + Nat.2) * seq_fib(Nat.2 * k + Nat.3) +
                seq_fib(Nat.2 * k + Nat.1) * seq_fib(Nat.2 * k + Nat.3)
        distrib_right(seq_fib(Nat.2 * k + Nat.2), seq_fib(Nat.2 * k + Nat.1), seq_fib(Nat.2 * k + Nat.3))
        seq_fib(Nat.2 * k + Nat.2) * seq_fib(Nat.2 * k + Nat.3) +
            seq_fib(Nat.2 * k + Nat.1) * seq_fib(Nat.2 * k + Nat.3) =
            (seq_fib(Nat.2 * k + Nat.2) + seq_fib(Nat.2 * k + Nat.1)) * seq_fib(Nat.2 * k + Nat.3)
        seq_fib_suc_suc(Nat.2 * k + Nat.1)
        seq_fib(Nat.2 * k + Nat.3) = seq_fib(Nat.2 * k + Nat.2) + seq_fib(Nat.2 * k + Nat.1)
        (seq_fib(Nat.2 * k + Nat.2) + seq_fib(Nat.2 * k + Nat.1)) * seq_fib(Nat.2 * k + Nat.3) =
            seq_fib(Nat.2 * k + Nat.3) * seq_fib(Nat.2 * k + Nat.3)
        seq_fib(Nat.2 * k + Nat.2) * seq_fib(Nat.2 * k + Nat.4) + Nat.1 =
            seq_fib(Nat.2 * k + Nat.3) * seq_fib(Nat.2 * k + Nat.3)
        mul_suc_right(Nat.2, k)
        Nat.2 * k.suc = Nat.2 * k + Nat.2
        Nat.2 * k.suc + Nat.1 = Nat.2 * k + Nat.3
        Nat.2 * k.suc + Nat.2 = Nat.2 * k + Nat.4
        seq_fib(Nat.2 * k.suc + Nat.1) * seq_fib(Nat.2 * k.suc + Nat.1) =
            seq_fib(Nat.2 * k.suc) * seq_fib(Nat.2 * k.suc + Nat.2) + Nat.1
    }
}

/// Cassini's identity, core statement: `F_{2k+1}^2 = F_{2k}F_{2k+2} + 1`.
///
/// By the classical identity `F_{n+1}F_{n-1} - F_n^2 = (-1)^n` this is the
/// statement at n = 2k + 1 (where the sign is -1, so
/// `F_n^2 = F_{n+1}F_{n-1} + 1`), and equivalently the square-difference
/// variant `F_{n+1}^2 - F_nF_{n+2} = (-1)^n` at n = 2k (sign +1).
theorem seq_fib_cassini_square(k: Nat) {
    seq_fib(Nat.2 * k + Nat.1) * seq_fib(Nat.2 * k + Nat.1) =
        seq_fib(Nat.2 * k) * seq_fib(Nat.2 * k + Nat.2) + Nat.1
} by {
    define p(j: Nat) -> Bool {
        seq_fib(Nat.2 * j + Nat.1) * seq_fib(Nat.2 * j + Nat.1) =
            seq_fib(Nat.2 * j) * seq_fib(Nat.2 * j + Nat.2) + Nat.1
    }
    seq_fib_zero
    seq_fib_one
    seq_fib_two
    seq_fib(Nat.1) * seq_fib(Nat.1) = seq_fib(Nat.0) * seq_fib(Nat.2) + Nat.1
    p(Nat.0)
    forall(j: Nat) {
        if p(j) {
            seq_fib_cassini_flip_a(j)
            seq_fib(Nat.2 * j + Nat.1) * seq_fib(Nat.2 * j + Nat.3) =
                seq_fib(Nat.2 * j + Nat.2) * seq_fib(Nat.2 * j + Nat.2) + Nat.1
            seq_fib_cassini_flip_b(j)
            seq_fib(Nat.2 * j.suc + Nat.1) * seq_fib(Nat.2 * j.suc + Nat.1) =
                seq_fib(Nat.2 * j.suc) * seq_fib(Nat.2 * j.suc + Nat.2) + Nat.1
            p(j.suc)
        }
    }
    p(Nat.0) and forall(j: Nat) { p(j) implies p(j.suc) }
    Nat.induction(p)
    p(k)
}

/// Cassini's identity, companion statement: `F_{2k+1}F_{2k+3} = F_{2k+2}^2 + 1`.
///
/// This is the classical identity at even n = 2k + 2 (sign +1), and the
/// square-difference variant at odd n = 2k + 1 (sign -1).
theorem seq_fib_cassini_product(k: Nat) {
    seq_fib(Nat.2 * k + Nat.1) * seq_fib(Nat.2 * k + Nat.3) =
        seq_fib(Nat.2 * k + Nat.2) * seq_fib(Nat.2 * k + Nat.2) + Nat.1
} by {
    seq_fib_cassini_square(k)
    seq_fib(Nat.2 * k + Nat.1) * seq_fib(Nat.2 * k + Nat.1) =
        seq_fib(Nat.2 * k) * seq_fib(Nat.2 * k + Nat.2) + Nat.1
    seq_fib_cassini_flip_a(k)
    seq_fib(Nat.2 * k + Nat.1) * seq_fib(Nat.2 * k + Nat.3) =
        seq_fib(Nat.2 * k + Nat.2) * seq_fib(Nat.2 * k + Nat.2) + Nat.1
}

/// Cassini's identity at even n = 2k (square-difference variant):
/// `F_{2k+1}^2 = F_{2k}F_{2k+2} + 1`.
theorem seq_fib_square_diff_even(k: Nat) {
    seq_fib(Nat.2 * k + Nat.1) * seq_fib(Nat.2 * k + Nat.1) =
        seq_fib(Nat.2 * k) * seq_fib(Nat.2 * k + Nat.2) + Nat.1
} by {
    seq_fib_cassini_square(k)
    seq_fib(Nat.2 * k + Nat.1) * seq_fib(Nat.2 * k + Nat.1) =
        seq_fib(Nat.2 * k) * seq_fib(Nat.2 * k + Nat.2) + Nat.1
}

/// Cassini's identity at odd n = 2k + 1 (square-difference variant):
/// `F_{2k+1}F_{2k+3} = F_{2k+2}^2 + 1`.
theorem seq_fib_square_diff_odd(k: Nat) {
    seq_fib(Nat.2 * k + Nat.1) * seq_fib(Nat.2 * k + Nat.3) =
        seq_fib(Nat.2 * k + Nat.2) * seq_fib(Nat.2 * k + Nat.2) + Nat.1
} by {
    seq_fib_cassini_product(k)
    seq_fib(Nat.2 * k + Nat.1) * seq_fib(Nat.2 * k + Nat.3) =
        seq_fib(Nat.2 * k + Nat.2) * seq_fib(Nat.2 * k + Nat.2) + Nat.1
}

/// Cassini's identity at odd n = 2k + 1:
/// `F_n^2 = F_{n+1}F_{n-1} + 1` for n = 2k + 1.
theorem seq_fib_cassini_odd(k: Nat) {
    seq_fib(Nat.2 * k + Nat.1) * seq_fib(Nat.2 * k + Nat.1) =
        seq_fib(Nat.2 * k + Nat.2) * seq_fib(Nat.2 * k) + Nat.1
} by {
    seq_fib_cassini_square(k)
    seq_fib(Nat.2 * k + Nat.1) * seq_fib(Nat.2 * k + Nat.1) =
        seq_fib(Nat.2 * k) * seq_fib(Nat.2 * k + Nat.2) + Nat.1
    mul_comm(seq_fib(Nat.2 * k), seq_fib(Nat.2 * k + Nat.2))
    seq_fib(Nat.2 * k) * seq_fib(Nat.2 * k + Nat.2) =
        seq_fib(Nat.2 * k + Nat.2) * seq_fib(Nat.2 * k)
    seq_fib(Nat.2 * k + Nat.1) * seq_fib(Nat.2 * k + Nat.1) =
        seq_fib(Nat.2 * k + Nat.2) * seq_fib(Nat.2 * k) + Nat.1
}

/// Cassini's identity at even n = 2k + 2:
/// `F_{n+1}F_{n-1} = F_n^2 + 1` for n = 2k + 2.
theorem seq_fib_cassini_even(k: Nat) {
    seq_fib(Nat.2 * k + Nat.3) * seq_fib(Nat.2 * k + Nat.1) =
        seq_fib(Nat.2 * k + Nat.2) * seq_fib(Nat.2 * k + Nat.2) + Nat.1
} by {
    seq_fib_cassini_product(k)
    seq_fib(Nat.2 * k + Nat.1) * seq_fib(Nat.2 * k + Nat.3) =
        seq_fib(Nat.2 * k + Nat.2) * seq_fib(Nat.2 * k + Nat.2) + Nat.1
    mul_comm(seq_fib(Nat.2 * k + Nat.1), seq_fib(Nat.2 * k + Nat.3))
    seq_fib(Nat.2 * k + Nat.1) * seq_fib(Nat.2 * k + Nat.3) =
        seq_fib(Nat.2 * k + Nat.3) * seq_fib(Nat.2 * k + Nat.1)
    seq_fib(Nat.2 * k + Nat.3) * seq_fib(Nat.2 * k + Nat.1) =
        seq_fib(Nat.2 * k + Nat.2) * seq_fib(Nat.2 * k + Nat.2) + Nat.1
}

// ---------------------------------------------------------------------------
// The addition formula (b): F_{m+n+1} = F_{m+1}F_{n+1} + F_mF_n.
//
// The induction on n carries the pair of statements
//   F_{m+n+1} = F_{m+1}F_{n+1} + F_mF_n  and  F_{m+n+2} = F_{m+1}F_{n+2} + F_mF_{n+1}
// because unfolding F_{m+n+2} by the recurrence needs both F_{m+n+1} and
// F_{m+n}.
// ---------------------------------------------------------------------------

/// The addition formula: `F_{m+n+1} = F_{m+1}F_{n+1} + F_mF_n` for all m, n.
///
/// With m replaced by m - 1 this is the classical formula
/// `F_{m+n} = F_mF_{n+1} + F_{m-1}F_n` (the shifted form avoids the index
/// m - 1, which is not available at m = 0).
theorem seq_fib_add(m: Nat, n: Nat) {
    seq_fib(m.suc + n) = seq_fib(m.suc) * seq_fib(n.suc) + seq_fib(m) * seq_fib(n)
} by {
    define p(j: Nat) -> Bool {
        seq_fib(m.suc + j) = seq_fib(m.suc) * seq_fib(j.suc) + seq_fib(m) * seq_fib(j)
            and
        seq_fib(m.suc + j.suc) =
            seq_fib(m.suc) * seq_fib(j.suc.suc) + seq_fib(m) * seq_fib(j.suc)
    }
    seq_fib_zero
    seq_fib(Nat.0) = Nat.0
    seq_fib_one
    seq_fib(Nat.1) = Nat.1
    seq_fib_two
    seq_fib(Nat.2) = Nat.1
    add_zero_right(m.suc)
    m.suc + Nat.0 = m.suc
    seq_fib(m.suc + Nat.0) = seq_fib(m.suc)
    seq_fib(Nat.0.suc) = Nat.1
    seq_fib(m.suc) * seq_fib(Nat.0.suc) + seq_fib(m) * seq_fib(Nat.0) =
        seq_fib(m.suc) * Nat.1 + seq_fib(m) * Nat.0
    mul_one_right(seq_fib(m.suc))
    seq_fib(m.suc) * Nat.1 = seq_fib(m.suc)
    mul_zero_right(seq_fib(m))
    seq_fib(m) * Nat.0 = Nat.0
    seq_fib(m.suc) * Nat.1 + seq_fib(m) * Nat.0 = seq_fib(m.suc) + Nat.0
    add_zero_right(seq_fib(m.suc))
    seq_fib(m.suc) + Nat.0 = seq_fib(m.suc)
    seq_fib(m.suc + Nat.0) = seq_fib(m.suc) * seq_fib(Nat.0.suc) + seq_fib(m) * seq_fib(Nat.0)
    seq_fib_suc_suc(m)
    seq_fib(m.suc.suc) = seq_fib(m.suc) + seq_fib(m)
    seq_fib(m.suc + Nat.0.suc) = seq_fib(m.suc) * seq_fib(Nat.0.suc.suc) + seq_fib(m) * seq_fib(Nat.0.suc)
    p(Nat.0)
    forall(j: Nat) {
        if p(j) {
            seq_fib(m.suc + j) = seq_fib(m.suc) * seq_fib(j.suc) + seq_fib(m) * seq_fib(j)
            seq_fib(m.suc + j.suc) =
                seq_fib(m.suc) * seq_fib(j.suc.suc) + seq_fib(m) * seq_fib(j.suc)
            seq_fib_suc_suc(m.suc + j)
            seq_fib((m.suc + j).suc.suc) = seq_fib((m.suc + j).suc) + seq_fib(m.suc + j)
            seq_fib(m.suc + j.suc.suc) = seq_fib(m.suc + j.suc) + seq_fib(m.suc + j)
            seq_fib(m.suc + j.suc) + seq_fib(m.suc + j) =
                (seq_fib(m.suc) * seq_fib(j.suc.suc) + seq_fib(m) * seq_fib(j.suc)) +
                (seq_fib(m.suc) * seq_fib(j.suc) + seq_fib(m) * seq_fib(j))
            seq_fib_suc_suc(j.suc)
            seq_fib(j.suc.suc.suc) = seq_fib(j.suc.suc) + seq_fib(j.suc)
            seq_fib_suc_suc(j)
            seq_fib(j.suc.suc) = seq_fib(j.suc) + seq_fib(j)
            seq_fib(m.suc) * seq_fib(j.suc.suc.suc) + seq_fib(m) * seq_fib(j.suc.suc) =
                seq_fib(m.suc) * (seq_fib(j.suc.suc) + seq_fib(j.suc)) +
                seq_fib(m) * (seq_fib(j.suc) + seq_fib(j))
            seq_fib(m.suc) * (seq_fib(j.suc.suc) + seq_fib(j.suc)) +
                seq_fib(m) * (seq_fib(j.suc) + seq_fib(j)) =
                (seq_fib(m.suc) * seq_fib(j.suc.suc) + seq_fib(m.suc) * seq_fib(j.suc)) +
                (seq_fib(m) * seq_fib(j.suc) + seq_fib(m) * seq_fib(j))
            seq_fib(m.suc) * seq_fib(j.suc.suc.suc) + seq_fib(m) * seq_fib(j.suc.suc) =
                (seq_fib(m.suc) * seq_fib(j.suc.suc) + seq_fib(m) * seq_fib(j.suc)) +
                (seq_fib(m.suc) * seq_fib(j.suc) + seq_fib(m) * seq_fib(j))
            seq_fib(m.suc + j.suc.suc) =
                seq_fib(m.suc) * seq_fib(j.suc.suc.suc) + seq_fib(m) * seq_fib(j.suc.suc)
            p(j.suc)
        }
    }
    p(Nat.0) and forall(j: Nat) { p(j) implies p(j.suc) }
    Nat.induction(p)
    p(n)
    seq_fib(m.suc + n) = seq_fib(m.suc) * seq_fib(n.suc) + seq_fib(m) * seq_fib(n)
}

// ---------------------------------------------------------------------------
// The parity pattern (e): F_n is even iff 3 | n.
//
// The full statement is stated below but not proved; the small cases
// n = 0..6 are verified, matching the pattern exactly.
// ---------------------------------------------------------------------------

/// The first Fibonacci numbers, for the small-case parity checks.
theorem seq_fib_three_value {
    seq_fib(Nat.3) = Nat.2
} by {
    seq_fib_suc_suc(Nat.1)
    seq_fib(Nat.3) = seq_fib(Nat.2) + seq_fib(Nat.1)
    seq_fib_two
    seq_fib_one
    seq_fib(Nat.3) = Nat.1 + Nat.1
    one_plus_one
    Nat.1 + Nat.1 = Nat.2
    seq_fib(Nat.3) = Nat.2
}

/// `F_4 = 3`.
theorem seq_fib_four_value {
    seq_fib(Nat.4) = Nat.3
} by {
    seq_fib_suc_suc(Nat.2)
    seq_fib(Nat.4) = seq_fib(Nat.3) + seq_fib(Nat.2)
    seq_fib_three_value
    seq_fib_two
    seq_fib(Nat.4) = Nat.2 + Nat.1
    Nat.2 + Nat.1 = Nat.3
    seq_fib(Nat.4) = Nat.3
}

/// `F_5 = 5`.
theorem seq_fib_five_value {
    seq_fib(Nat.5) = Nat.5
} by {
    seq_fib_suc_suc(Nat.3)
    seq_fib(Nat.5) = seq_fib(Nat.4) + seq_fib(Nat.3)
    seq_fib_four_value
    seq_fib_three_value
    seq_fib(Nat.5) = Nat.3 + Nat.2
    Nat.3 + Nat.2 = Nat.5
    seq_fib(Nat.5) = Nat.5
}

/// `F_6 = 8`.
theorem seq_fib_six_value {
    seq_fib(Nat.6) = Nat.8
} by {
    seq_fib_suc_suc(Nat.4)
    seq_fib(Nat.6) = seq_fib(Nat.5) + seq_fib(Nat.4)
    seq_fib_five_value
    seq_fib_four_value
    seq_fib(Nat.6) = Nat.5 + Nat.3
    Nat.5 + Nat.3 = Nat.8
    seq_fib(Nat.6) = Nat.8
}

/// `F_0 = 0` is even.
theorem seq_fib_zero_even {
    Nat.2.divides(seq_fib(Nat.0))
} by {
    seq_fib_zero
    seq_fib(Nat.0) = Nat.0
    divides_zero(Nat.2)
    Nat.2.divides(Nat.0)
    Nat.2.divides(seq_fib(Nat.0))
}

/// `F_1 = 1` is odd.
theorem seq_fib_one_odd {
    not Nat.2.divides(seq_fib(Nat.1))
} by {
    seq_fib_one
    seq_fib(Nat.1) = Nat.1
    two_divides_suc_iff(Nat.0)
    Nat.2.divides(Nat.1) = not Nat.2.divides(Nat.0)
    divides_zero(Nat.2)
    Nat.2.divides(Nat.0)
    Nat.2.divides(Nat.1) = not true
    not Nat.2.divides(Nat.1)
    not Nat.2.divides(seq_fib(Nat.1))
}

/// `F_2 = 1` is odd.
theorem seq_fib_two_odd {
    not Nat.2.divides(seq_fib(Nat.2))
} by {
    seq_fib_two
    seq_fib(Nat.2) = Nat.1
    two_divides_suc_iff(Nat.0)
    Nat.2.divides(Nat.1) = not Nat.2.divides(Nat.0)
    divides_zero(Nat.2)
    Nat.2.divides(Nat.0)
    Nat.2.divides(Nat.1) = not true
    not Nat.2.divides(Nat.1)
    not Nat.2.divides(seq_fib(Nat.2))
}

/// `F_3 = 2` is even.
theorem seq_fib_three_even {
    Nat.2.divides(seq_fib(Nat.3))
} by {
    seq_fib_three_value
    seq_fib(Nat.3) = Nat.2
    Nat.2 * Nat.1 = Nat.2
    exists(c: Nat) { Nat.2 * c = Nat.2 }
    Nat.2.divides(Nat.2)
    Nat.2.divides(seq_fib(Nat.3))
}

/// `F_4 = 3` is odd.
theorem seq_fib_four_odd {
    not Nat.2.divides(seq_fib(Nat.4))
} by {
    seq_fib_four_value
    seq_fib(Nat.4) = Nat.3
    if Nat.2.divides(Nat.3) {
        div_imp_mod(Nat.3, Nat.2)
        Nat.3.mod(Nat.2) = Nat.0
        mod_add_mul(Nat.1, Nat.2, Nat.1)
        (Nat.1 * Nat.2 + Nat.1).mod(Nat.2) = Nat.1.mod(Nat.2)
        Nat.1 * Nat.2 + Nat.1 = Nat.3
        Nat.3.mod(Nat.2) = Nat.1.mod(Nat.2)
        lt_suc(Nat.1)
        Nat.1 < Nat.2
        small_mod(Nat.1, Nat.2)
        Nat.1.mod(Nat.2) = Nat.1
        Nat.3.mod(Nat.2) = Nat.1
        Nat.0 = Nat.1
        suc_ne(Nat.0)
        Nat.1 != Nat.0
        false
    }
    not Nat.2.divides(Nat.3)
    not Nat.2.divides(seq_fib(Nat.4))
}

/// `F_5 = 5` is odd.
theorem seq_fib_five_odd {
    not Nat.2.divides(seq_fib(Nat.5))
} by {
    seq_fib_five_value
    seq_fib(Nat.5) = Nat.5
    if Nat.2.divides(Nat.5) {
        div_imp_mod(Nat.5, Nat.2)
        Nat.5.mod(Nat.2) = Nat.0
        mod_add_mul(Nat.2, Nat.2, Nat.1)
        (Nat.2 * Nat.2 + Nat.1).mod(Nat.2) = Nat.1.mod(Nat.2)
        Nat.2 * Nat.2 + Nat.1 = Nat.5
        Nat.5.mod(Nat.2) = Nat.1.mod(Nat.2)
        lt_suc(Nat.1)
        Nat.1 < Nat.2
        small_mod(Nat.1, Nat.2)
        Nat.1.mod(Nat.2) = Nat.1
        Nat.5.mod(Nat.2) = Nat.1
        Nat.0 = Nat.1
        suc_ne(Nat.0)
        Nat.1 != Nat.0
        false
    }
    not Nat.2.divides(Nat.5)
    not Nat.2.divides(seq_fib(Nat.5))
}

/// `F_6 = 8` is even.
theorem seq_fib_six_even {
    Nat.2.divides(seq_fib(Nat.6))
} by {
    seq_fib_six_value
    seq_fib(Nat.6) = Nat.8
    mul_two_left(Nat.4)
    Nat.2 * Nat.4 = Nat.4 + Nat.4
    Nat.4 + Nat.4 = Nat.8
    Nat.2 * Nat.4 = Nat.8
    exists(c: Nat) { Nat.2 * c = Nat.8 }
    Nat.2.divides(Nat.8)
    Nat.2.divides(seq_fib(Nat.6))
}

/// The index n = 0 satisfies `3 | 0`, matching `F_0` even.
theorem three_divides_zero {
    Nat.3.divides(Nat.0)
} by {
    divides_zero(Nat.3)
    Nat.3.divides(Nat.0)
}

/// The index n = 3 satisfies `3 | 3`, matching `F_3` even.
theorem three_divides_three {
    Nat.3.divides(Nat.3)
} by {
    divides_self(Nat.3)
    Nat.3.divides(Nat.3)
}

/// The index n = 6 satisfies `3 | 6`, matching `F_6` even.
theorem three_divides_six {
    Nat.3.divides(Nat.6)
} by {
    mul_comm(Nat.3, Nat.2)
    Nat.3 * Nat.2 = Nat.2 * Nat.3
    mul_two_left(Nat.3)
    Nat.2 * Nat.3 = Nat.3 + Nat.3
    Nat.3 + Nat.3 = Nat.6
    Nat.3 * Nat.2 = Nat.6
    exists(c: Nat) { Nat.3 * c = Nat.6 }
    Nat.3.divides(Nat.6)
}

/// The index n = 1 does not satisfy `3 | 1`, matching `F_1` odd.
theorem three_not_divides_one {
    not Nat.3.divides(Nat.1)
} by {
    if Nat.3.divides(Nat.1) {
        div_imp_mod(Nat.1, Nat.3)
        Nat.1.mod(Nat.3) = Nat.0
        lt_suc(Nat.1)
        Nat.1 < Nat.2
        lt_imp_lt_suc(Nat.1, Nat.2)
        Nat.1 < Nat.3
        small_mod(Nat.1, Nat.3)
        Nat.1.mod(Nat.3) = Nat.1
        Nat.0 = Nat.1
        suc_ne(Nat.0)
        Nat.1 != Nat.0
        false
    }
    not Nat.3.divides(Nat.1)
}

/// The index n = 2 does not satisfy `3 | 2`, matching `F_2` odd.
theorem three_not_divides_two {
    not Nat.3.divides(Nat.2)
} by {
    if Nat.3.divides(Nat.2) {
        div_imp_mod(Nat.2, Nat.3)
        Nat.2.mod(Nat.3) = Nat.0
        lt_suc(Nat.2)
        Nat.2 < Nat.3
        small_mod(Nat.2, Nat.3)
        Nat.2.mod(Nat.3) = Nat.2
        Nat.0 = Nat.2
        Nat.2 != Nat.0
        false
    }
    not Nat.3.divides(Nat.2)
}

/// The index n = 4 does not satisfy `3 | 4`, matching `F_4` odd.
theorem three_not_divides_four {
    not Nat.3.divides(Nat.4)
} by {
    if Nat.3.divides(Nat.4) {
        div_imp_mod(Nat.4, Nat.3)
        Nat.4.mod(Nat.3) = Nat.0
        mod_add_mul(Nat.1, Nat.3, Nat.1)
        (Nat.1 * Nat.3 + Nat.1).mod(Nat.3) = Nat.1.mod(Nat.3)
        Nat.1 * Nat.3 + Nat.1 = Nat.4
        Nat.4.mod(Nat.3) = Nat.1.mod(Nat.3)
        lt_suc(Nat.1)
        Nat.1 < Nat.2
        lt_imp_lt_suc(Nat.1, Nat.2)
        Nat.1 < Nat.3
        small_mod(Nat.1, Nat.3)
        Nat.1.mod(Nat.3) = Nat.1
        Nat.4.mod(Nat.3) = Nat.1
        Nat.0 = Nat.1
        suc_ne(Nat.0)
        Nat.1 != Nat.0
        false
    }
    not Nat.3.divides(Nat.4)
}

/// The index n = 5 does not satisfy `3 | 5`, matching `F_5` odd.
theorem three_not_divides_five {
    not Nat.3.divides(Nat.5)
} by {
    if Nat.3.divides(Nat.5) {
        div_imp_mod(Nat.5, Nat.3)
        Nat.5.mod(Nat.3) = Nat.0
        mod_add_mul(Nat.1, Nat.3, Nat.2)
        (Nat.1 * Nat.3 + Nat.2).mod(Nat.3) = Nat.2.mod(Nat.3)
        Nat.1 * Nat.3 + Nat.2 = Nat.5
        Nat.5.mod(Nat.3) = Nat.2.mod(Nat.3)
        lt_suc(Nat.2)
        Nat.2 < Nat.3
        small_mod(Nat.2, Nat.3)
        Nat.2.mod(Nat.3) = Nat.2
        Nat.5.mod(Nat.3) = Nat.2
        Nat.0 = Nat.2
        Nat.2 != Nat.0
        false
    }
    not Nat.3.divides(Nat.5)
}

// ---------------------------------------------------------------------------
// The parity pattern in full: F_n is even iff 3 divides n.
//
// The statement is:
//
//   theorem seq_fib_even_iff_three_divides(n: Nat) {
//       Nat.2.divides(seq_fib(n)) = Nat.3.divides(n)
//   }
//
// It is left unproved here (as is the gcd identity below): the forward
// direction needs a triple induction on the repeating pattern (even, odd,
// odd) of (F_{3k}, F_{3k+1}, F_{3k+2}) together with the parity algebra of
// sums of even and odd naturals; the small cases n = 0..6 above already
// exhibit the pattern.
// ---------------------------------------------------------------------------

// ---------------------------------------------------------------------------
// The gcd identity (d): gcd(F_m, F_n) = F_{gcd(m,n)}.
//
// The statement is:
//
//   theorem seq_fib_gcd(m: Nat, n: Nat) {
//       seq_fib(m).gcd(seq_fib(n)) = seq_fib(m.gcd(n))
//   }
//
// It is left unproved here.  A proof would need the Euclidean-descent facts
// gcd(F_m, F_n) = gcd(F_m, F_{n mod m}) (via the addition formula) and the
// induction on the Euclidean algorithm, which is beyond the scope of this
// file.
// ---------------------------------------------------------------------------
