from number_theory.coprime import Nat, coprime_one_left, coprime_mul_iff, coprime_mul, coprime_comm, coprime_mod_imp,
    coprime_zero_left_imp_one, coprime_zero_right_imp_one
from nat import gcd_zero_left, gcd_of_prime, gcd_comm, divides_gcd
from number_theory.factorisation import prime_divisor_is_one_or_self, coprime_of_distinct_primes
from nat import divides_self, divides_zero, add_imp_sub, small_mod, mod_mod, add_mod,
    add_cancels_left, mod_by_zero, add_one_right, lte_ref, lte_trans,
    lt_cancel_mul, lte_mul, lt_diff, alt_add_zero, not_lt_zero, mul_one_left,
    mod_lte, alt_induction, sum_lte
from number_theory.crt import nat_coprime_combine, nat_congr_combine_coprime
from number_theory.fermat import rsa_pow_congr
from number_theory.modular_inverse import cancel_coprime, mod_inv, mod_inv_coprime, mod_inv_mul_congr_one
from nat import exp_one, exp_add, exp_zero
from number_theory.coprime import coprime_one_right, coprime_mod_iff
from number_theory.congruence import congr_mod_refl, congr_mod_symm, congr_mod_trans, congr_mod_mul, congr_mod_add, mod_lt, mod_congr_mod_self, mod_add_mul, mod_add_eq
from list import List, list_contains_implies_count_geq_one, list_not_contains_impl_count_zero,
    unique_implies_no_duplicate, singleton_unique, unique_list_sum,
    unique_implies_tail_unique, filter_contained_by_and, filter_equivalent_to_and
from list import map, map_length, map_contains, map_contains_of_contains
from list import sum
from list import is_permutation, permutation_preserves_length, permutation_preserves_product,
    unique_same_contains_imp_permutation
from list import length_range, range_contains_iff_lt, range_does_not_contain_geq,
    filter_length_of_pointwise, map_filter_length_of_pointwise
from data.finite.finite_fiber_partition import locally_injective_map_is_unique
from list import product
from data.list.list_pair_product import list_pair_product, list_pair_product_filter_length_sum_rows,
    list_pair_product_row_filter_length_fn, list_pair_with_left, list_pair_with_left_filter_length,
    row_encode, row_encode_filter_pred, row_encode_filter_length_range,
    filter_indicator_value, sum_filter_indicator_value
from pair import Pair
from data.basic.functions import function_extensionality
numerals Nat

/// The number of natural numbers in `[0, k)` that are coprime to `n`. Used to
/// build Euler's totient via `nat_totient(n) = count_coprime_to(n, n)`.
define count_coprime_to(n: Nat, k: Nat) -> Nat {
    match k {
        Nat.zero {
            Nat.0
        }
        Nat.suc(j) {
            if j.coprime(n) {
                count_coprime_to(n, j) + Nat.1
            } else {
                count_coprime_to(n, j)
            }
        }
    }
}

attributes Nat {
    /// Euler's totient: the number of natural numbers in `[0, n)` that are
    /// coprime to `n`. Reduces to `n - 1` for prime `n` and to
    /// `(p - 1) * (q - 1)` for distinct primes `p`, `q`.
    define totient(self) -> Nat {
        count_coprime_to(self, self)
    }
}

/// Euler's totient as an arithmetic function.
let nat_totient: Nat -> Nat = function(n: Nat) { n.totient }

/// Recurrence: `count_coprime_to(n, k.suc)` is `count_coprime_to(n, k) + 1` when
/// `k` is coprime to `n`.
theorem count_coprime_to_suc_yes(n: Nat, k: Nat) {
    k.coprime(n)
        implies count_coprime_to(n, k.suc) = count_coprime_to(n, k) + Nat.1
}

/// Recurrence: `count_coprime_to(n, k.suc)` equals `count_coprime_to(n, k)` when
/// `k` is not coprime to `n`.
theorem count_coprime_to_suc_no(n: Nat, k: Nat) {
    not k.coprime(n)
        implies count_coprime_to(n, k.suc) = count_coprime_to(n, k)
}

/// Euler's totient vanishes at zero.
theorem nat_totient_zero {
    nat_totient(Nat.0) = Nat.0
} by {
    nat_totient(Nat.0) = Nat.0.totient
    Nat.0.totient = count_coprime_to(Nat.0, Nat.0)
    count_coprime_to(Nat.0, Nat.0) = Nat.0
    nat_totient(Nat.0) = Nat.0
}

/// Euler's totient is one at one.
theorem nat_totient_one {
    nat_totient(Nat.1) = Nat.1
} by {
    coprime_one_right(Nat.0)
    Nat.0.coprime(Nat.1)
    count_coprime_to_suc_yes(Nat.1, Nat.0)
    count_coprime_to(Nat.1, Nat.1) = count_coprime_to(Nat.1, Nat.0) + Nat.1
    count_coprime_to(Nat.1, Nat.0) = Nat.0
    count_coprime_to(Nat.1, Nat.1) = Nat.1
    nat_totient(Nat.1) = Nat.1.totient
    Nat.1.totient = count_coprime_to(Nat.1, Nat.1)
    nat_totient(Nat.1) = Nat.1
}

/// Inductive predicate for `totient_prime`: `count_coprime_to(p, k)` equals
/// `k - 1` for `1 <= k <= p` when `p` is prime.
define totient_prime_pred(p: Nat) -> (Nat -> Bool) {
    function(k: Nat) {
        Nat.1 <= k and k <= p implies count_coprime_to(p, k) = k - Nat.1
    }
}

/// Base case for the prime-totient recurrence: `count_coprime_to(p, 1) = 0`.
theorem totient_prime_base(p: Nat) {
    p.is_prime implies totient_prime_pred(p)(Nat.1)
} by {
    if p.is_prime {
        if Nat.1 <= Nat.1 and Nat.1 <= p {
            // Nat.0 not coprime to prime p (gcd(0, p) = p != 1).
            gcd_zero_left(p)
            Nat.0.gcd(p) = p
            Nat.1 < p
            p != Nat.1
            not Nat.0.coprime(p)
            count_coprime_to_suc_no(p, Nat.0)
            count_coprime_to(p, Nat.1) = count_coprime_to(p, Nat.0)
            count_coprime_to(p, Nat.0) = Nat.0
            count_coprime_to(p, Nat.1) = Nat.0
            Nat.1 - Nat.1 = Nat.0
            count_coprime_to(p, Nat.1) = Nat.1 - Nat.1
        }
    }
}

/// For prime `p` and `1 <= k < p`, `k` is coprime to `p`.
theorem coprime_below_prime(p: Nat, k: Nat) {
    p.is_prime and Nat.1 <= k and k < p implies k.coprime(p)
} by {
    if p.is_prime and Nat.1 <= k and k < p {
        // gcd(k, p) divides p prime, so it's 1 or p; cannot be p since k < p.
        let d: Nat = k.gcd(p)
        // d divides p (gcd_divides_right gives this).
        // d divides k as well, so d <= k < p (so d != p).
        // d > 0 since p > 0.
        d.divides(p)
        d.divides(k)
        prime_divisor_is_one_or_self(p, d)
        d = Nat.1 or d = p
        if d = p {
            // p divides k, but k < p and k > 0 contradicts.
            p.divides(k)
            k != Nat.0
            // p <= k contradicts k < p.
            false
        }
        d = Nat.1
        k.gcd(p) = Nat.1
        k.coprime(p)
    }
}

/// Inductive step for the prime-totient recurrence.
theorem totient_prime_step(p: Nat, k: Nat) {
    p.is_prime and totient_prime_pred(p)(k) implies totient_prime_pred(p)(k.suc)
} by {
    if p.is_prime and totient_prime_pred(p)(k) {
        if Nat.1 <= k.suc and k.suc <= p {
            if k = Nat.0 {
                // Reduces to the base case.
                totient_prime_base(p)
                totient_prime_pred(p)(Nat.1)
                Nat.1 <= Nat.1
                Nat.1 <= p
                count_coprime_to(p, Nat.1) = Nat.1 - Nat.1
                k.suc = Nat.1
                k.suc - Nat.1 = Nat.1 - Nat.1
                count_coprime_to(p, k.suc) = k.suc - Nat.1
            } else {
                Nat.1 <= k
                k <= k.suc
                k.suc <= p
                k <= p
                count_coprime_to(p, k) = k - Nat.1
                // k is in [1, p) since k < k.suc <= p.
                k < k.suc
                k <= p
                k != p or k = p
                if k = p {
                    // k.suc = p.suc > p contradicts k.suc <= p.
                    false
                }
                k != p
                k < p
                coprime_below_prime(p, k)
                k.coprime(p)
                count_coprime_to_suc_yes(p, k)
                count_coprime_to(p, k.suc) = count_coprime_to(p, k) + Nat.1
                count_coprime_to(p, k.suc) = (k - Nat.1) + Nat.1
                // k - 1 + 1 = k since k >= 1.
                Nat.1 <= k
                let kp: Nat satisfy { kp.suc = k }
                k - Nat.1 = kp
                kp + Nat.1 = kp.suc
                (k - Nat.1) + Nat.1 = k
                count_coprime_to(p, k.suc) = k
                // Goal: count_coprime_to(p, k.suc) = k.suc - 1 = k.
                k.suc - Nat.1 = k
                count_coprime_to(p, k.suc) = k.suc - Nat.1
            }
        }
    }
}

/// Inner induction for `totient_prime`: walks the recurrence up to `n = p`.
theorem totient_prime_run(p: Nat, n: Nat) {
    p.is_prime implies totient_prime_pred(p)(n)
} by {
    if p.is_prime {
        let f: Nat -> Bool = function(x: Nat) {
            Nat.1 <= x and x <= p implies count_coprime_to(p, x) = x - Nat.1
        }
        forall(y: Nat) {
            totient_prime_pred(p)(y) = f(y)
            f(y) = totient_prime_pred(p)(y)
        }
        // Base at 0: vacuous since not (1 <= 0).
        if Nat.1 <= Nat.0 {
            false
        }
        not (Nat.1 <= Nat.0)
        f(Nat.0)
        forall(x: Nat) {
            if f(x) {
                totient_prime_pred(p)(x)
                totient_prime_step(p, x)
                totient_prime_pred(p)(x.suc)
                f(x.suc)
            }
        }
        f(n)
        totient_prime_pred(p)(n)
    }
}

/// Euler's totient at a prime: `totient(p) = p - 1`.
theorem totient_prime(p: Nat) {
    p.is_prime implies p.totient = p - Nat.1
} by {
    if p.is_prime {
        totient_prime_run(p, p)
        totient_prime_pred(p)(p)
        Nat.1 <= p
        p <= p
        count_coprime_to(p, p) = p - Nat.1
        p.totient = count_coprime_to(p, p)
        p.totient = p - Nat.1
    }
}

/// `n - 1 = pred` whenever `pred.suc = n`. Local helper to avoid pulling in
/// `nat_fermat`.
theorem totient_sub_one_pred(n: Nat, pred: Nat) {
    pred.suc = n implies n - Nat.1 = pred
} by {
    if pred.suc = n {
        pred.suc = pred + Nat.1
        pred + Nat.1 = n
        add_imp_sub(pred, Nat.1, n)
    }
}

/// The number of natural numbers in `[0, k)` divisible by `d`.
/// Used for inclusion-exclusion-style counts that feed into totient identities.
define count_multiples(d: Nat, k: Nat) -> Nat {
    match k {
        Nat.zero {
            Nat.0
        }
        Nat.suc(j) {
            if d.divides(j) {
                count_multiples(d, j) + Nat.1
            } else {
                count_multiples(d, j)
            }
        }
    }
}

/// Recurrence: stepping `k` past a multiple of `d` adds one to the count.
theorem count_multiples_suc_yes(d: Nat, k: Nat) {
    d.divides(k) implies count_multiples(d, k.suc) = count_multiples(d, k) + Nat.1
}

/// Recurrence: stepping `k` past a non-multiple of `d` leaves the count alone.
theorem count_multiples_suc_no(d: Nat, k: Nat) {
    not d.divides(k) implies count_multiples(d, k.suc) = count_multiples(d, k)
}

/// Within an aligned block, `d` divides `k + j` only when `j = 0`. For
/// `0 < j < d`, the offset `j` is too small to bridge two multiples of `d`.
theorem count_multiples_block_no_div(d: Nat, k: Nat, j: Nat) {
    d != Nat.0 and d.divides(k) and Nat.0 < j and j < d
        implies not d.divides(k + j)
} by {
    if d != Nat.0 and d.divides(k) and Nat.0 < j and j < d {
        if d.divides(k + j) {
            let qq: Nat satisfy { d * qq = k + j }
            let qp: Nat satisfy { d * qp = k }
            d * qq = d * qp + j
            // qp < qq because d * qp < d * qp + j = d * qq.
            d * qp < d * qp + j
            d * qp < d * qq
            qp < qq
            let dq: Nat satisfy { qp + dq = qq and dq != Nat.0 }
            d * (qp + dq) = d * qp + d * dq
            d * qp + d * dq = d * qp + j
            d * dq = j
            d <= d * dq
            d <= j
            false
        }
    }
}

/// Inductive predicate for `count_multiples_block_aligned`. It tracks the
/// offset `j` from the starting multiple `k`. For any `j` in `[0, d]`, the
/// count of multiples of `d` in `[0, k + j)` exceeds the count in `[0, k)`
/// by `0` if `j = 0` and by `1` otherwise.
define block_aligned_pred(d: Nat, k: Nat) -> (Nat -> Bool) {
    function(j: Nat) {
        j <= d and Nat.0 < j
            implies count_multiples(d, k + j) = count_multiples(d, k) + Nat.1
    }
}

/// Inductive step for the block-aligned recurrence.
theorem block_aligned_step(d: Nat, k: Nat, j: Nat) {
    d != Nat.0 and d.divides(k) and block_aligned_pred(d, k)(j)
        implies block_aligned_pred(d, k)(j.suc)
} by {
    if d != Nat.0 and d.divides(k) and block_aligned_pred(d, k)(j) {
        if j.suc <= d and Nat.0 < j.suc {
            j <= d
            j < d
            k + j.suc = (k + j).suc
            if j = Nat.0 {
                // Stepping past k itself adds one to the count.
                k + j = k
                d.divides(k + j)
                count_multiples_suc_yes(d, k + j)
                count_multiples(d, (k + j).suc) = count_multiples(d, k + j) + Nat.1
                k + j = k
                count_multiples(d, k + j) = count_multiples(d, k)
                count_multiples(d, k + j.suc) = count_multiples(d, k) + Nat.1
            } else {
                Nat.0 < j
                count_multiples(d, k + j) = count_multiples(d, k) + Nat.1
                count_multiples_block_no_div(d, k, j)
                not d.divides(k + j)
                count_multiples_suc_no(d, k + j)
                count_multiples(d, (k + j).suc) = count_multiples(d, k + j)
                count_multiples(d, k + j.suc) = count_multiples(d, k) + Nat.1
            }
        }
        j.suc != Nat.0
        j.suc <= d and Nat.0 < j.suc implies count_multiples(d, k + j.suc) = count_multiples(d, k) + Nat.1
        block_aligned_pred(d, k)(j.suc) =
            (j.suc <= d and Nat.0 < j.suc implies count_multiples(d, k + j.suc) = count_multiples(d, k) + Nat.1)
        block_aligned_pred(d, k)(j.suc)
    }
}

/// Walking `d` steps forward through `count_multiples(d, _)` adds exactly one
/// when `d > 0` and `d` divides the starting index. The next multiple is the
/// only multiple of `d` in `[k, k + d)`, which produces the increment.
theorem count_multiples_block_aligned(d: Nat, k: Nat) {
    d != Nat.0 and d.divides(k)
        implies count_multiples(d, k + d) = count_multiples(d, k) + Nat.1
} by {
    if d != Nat.0 and d.divides(k) {
        let g: Nat -> Bool = function(j: Nat) {
            j <= d and Nat.0 < j implies count_multiples(d, k + j) = count_multiples(d, k) + Nat.1
        }
        forall(y: Nat) {
            block_aligned_pred(d, k)(y) = g(y)
            g(y) = block_aligned_pred(d, k)(y)
        }
        // Base: vacuous since not (0 < 0).
        if Nat.0 <= d and Nat.0 < Nat.0 {
            false
        }
        g(Nat.0)
        // Step.
        forall(j: Nat) {
            if g(j) {
                block_aligned_pred(d, k)(j)
                block_aligned_step(d, k, j)
                block_aligned_pred(d, k)(j.suc)
                g(j.suc)
            }
        }
        // Conclude at j = d.
        Nat.0 < d
        d <= d
        g(d)
        g(d) = (d <= d and Nat.0 < d implies count_multiples(d, k + d) = count_multiples(d, k) + Nat.1)
        d <= d and Nat.0 < d implies count_multiples(d, k + d) = count_multiples(d, k) + Nat.1
        count_multiples(d, k + d) = count_multiples(d, k) + Nat.1
    }
}

/// Predicate version of `count_multiples_div`.
define count_multiples_div_pred(d: Nat) -> (Nat -> Bool) {
    function(q: Nat) {
        d != Nat.0 implies count_multiples(d, d * q) = q
    }
}

/// Walking `q` aligned blocks of length `d` starting from zero gives count `q`.
theorem count_multiples_div(d: Nat, q: Nat) {
    d != Nat.0 implies count_multiples(d, d * q) = q
} by {
    let f: Nat -> Bool = function(x: Nat) {
        d != Nat.0 implies count_multiples(d, d * x) = x
    }
    forall(y: Nat) {
        count_multiples_div_pred(d)(y) = f(y)
        f(y) = count_multiples_div_pred(d)(y)
    }
    // Base.
    d * Nat.0 = Nat.0
    count_multiples(d, Nat.0) = Nat.0
    if d != Nat.0 {
        count_multiples(d, d * Nat.0) = Nat.0
    }
    f(Nat.0)
    // Step.
    forall(x: Nat) {
        if f(x) {
            if d != Nat.0 {
                count_multiples(d, d * x) = x
                // d * x.suc = d * x + d.
                d * x.suc = d * x + d
                // d divides d * x.
                d.divides(d * x)
                count_multiples_block_aligned(d, d * x)
                count_multiples(d, d * x + d) = count_multiples(d, d * x) + Nat.1
                count_multiples(d, d * x.suc) = x + Nat.1
                count_multiples(d, d * x.suc) = x.suc
            }
            f(x.suc)
        }
    }
    f(q)
}

/// Counterpart to `count_coprime_to`: the number of natural numbers in
/// `[0, k)` that are NOT coprime to `n`. Together with `count_coprime_to` it
/// partitions `[0, k)`.
define count_not_coprime_to(n: Nat, k: Nat) -> Nat {
    match k {
        Nat.zero {
            Nat.0
        }
        Nat.suc(j) {
            if j.coprime(n) {
                count_not_coprime_to(n, j)
            } else {
                count_not_coprime_to(n, j) + Nat.1
            }
        }
    }
}

/// Inductive predicate for the coprime/non-coprime conservation law.
define coprime_partition_pred(n: Nat) -> (Nat -> Bool) {
    function(k: Nat) {
        count_coprime_to(n, k) + count_not_coprime_to(n, k) = k
    }
}

/// Conservation: the coprime and not-coprime counts in `[0, k)` add up to `k`.
theorem coprime_partition(n: Nat, k: Nat) {
    count_coprime_to(n, k) + count_not_coprime_to(n, k) = k
} by {
    let f: Nat -> Bool = function(x: Nat) {
        count_coprime_to(n, x) + count_not_coprime_to(n, x) = x
    }
    forall(y: Nat) {
        coprime_partition_pred(n)(y) = f(y)
        f(y) = coprime_partition_pred(n)(y)
    }
    // Base.
    count_coprime_to(n, Nat.0) = Nat.0
    count_not_coprime_to(n, Nat.0) = Nat.0
    f(Nat.0)
    // Step.
    forall(x: Nat) {
        if f(x) {
            count_coprime_to(n, x) + count_not_coprime_to(n, x) = x
            let a: Nat = count_coprime_to(n, x)
            let b: Nat = count_not_coprime_to(n, x)
            a + b = x
            if x.coprime(n) {
                count_coprime_to(n, x.suc) = a + Nat.1
                count_not_coprime_to(n, x.suc) = b
                count_coprime_to(n, x.suc) + count_not_coprime_to(n, x.suc) = (a + Nat.1) + b
                (a + Nat.1) + b = a + (Nat.1 + b)
                Nat.1 + b = b + Nat.1
                a + (Nat.1 + b) = a + (b + Nat.1)
                a + (b + Nat.1) = (a + b) + Nat.1
                (a + Nat.1) + b = (a + b) + Nat.1
                count_coprime_to(n, x.suc) + count_not_coprime_to(n, x.suc) = x + Nat.1
                count_coprime_to(n, x.suc) + count_not_coprime_to(n, x.suc) = x.suc
            } else {
                count_coprime_to(n, x.suc) = a
                count_not_coprime_to(n, x.suc) = b + Nat.1
                count_coprime_to(n, x.suc) + count_not_coprime_to(n, x.suc) = a + (b + Nat.1)
                a + (b + Nat.1) = (a + b) + Nat.1
                count_coprime_to(n, x.suc) + count_not_coprime_to(n, x.suc) = x + Nat.1
                count_coprime_to(n, x.suc) + count_not_coprime_to(n, x.suc) = x.suc
            }
            f(x.suc)
        }
    }
    f(k)
}

/// Forward: if `k` is not coprime to prime `p`, then `p` divides `k`.
theorem not_coprime_imp_divides_prime(p: Nat, k: Nat) {
    p.is_prime and not k.coprime(p) implies p.divides(k)
} by {
    if p.is_prime and not k.coprime(p) {
        gcd_of_prime(p, k)
        if p.gcd(k) = Nat.1 {
            gcd_comm(p, k)
            k.gcd(p) = Nat.1
            k.coprime(p)
            false
        }
    }
}

/// Backward: if prime `p` divides `k`, then `k` is not coprime to `p`.
theorem divides_prime_imp_not_coprime(p: Nat, k: Nat) {
    p.is_prime and p.divides(k) implies not k.coprime(p)
} by {
    if p.is_prime and p.divides(k) {
        if k.coprime(p) {
            gcd_comm(k, p)
            k.gcd(p) = p.gcd(k)
            p.gcd(k) = Nat.1
            divides_gcd(p, p, k)
            p.divides(p.gcd(k))
            p.divides(Nat.1)
            let kk: Nat satisfy { p * kk = Nat.1 }
            p = Nat.1
            Nat.1 < p
            false
        }
    }
}

/// For prime `p`, "not coprime to `p`" and "`p` divides" agree.
theorem not_coprime_iff_divides_prime(p: Nat, k: Nat) {
    p.is_prime implies (not k.coprime(p) = p.divides(k))
} by {
    if p.is_prime {
        if not k.coprime(p) {
            not_coprime_imp_divides_prime(p, k)
        }
        if p.divides(k) {
            divides_prime_imp_not_coprime(p, k)
        }
    }
}

/// Forward: `m * n` divides `k` implies both `m` and `n` divide `k`.
theorem divides_mul_imp_each(m: Nat, n: Nat, k: Nat) {
    (m * n).divides(k) implies m.divides(k) and n.divides(k)
} by {
    if (m * n).divides(k) {
        let c: Nat satisfy { m * n * c = k }
        m * n * c = m * (n * c)
        k = m * (n * c)
        m.divides(k)
        m * n * c = n * (m * c)
        k = n * (m * c)
        n.divides(k)
    }
}


/// Forward: if `k` is not coprime to `p * q` for distinct primes `p`, `q`, then
/// `p` or `q` divides `k`.
theorem not_coprime_pq_imp_div(p: Nat, q: Nat, k: Nat) {
    p.is_prime and q.is_prime and p != q and not k.coprime(p * q)
        implies p.divides(k) or q.divides(k)
} by {
    if p.is_prime and q.is_prime and p != q and not k.coprime(p * q) {
        coprime_of_distinct_primes(p, q)
        p.coprime(q)
        coprime_mul_iff(k, p, q)
        not (k.coprime(p) and k.coprime(q))
        not k.coprime(p) or not k.coprime(q)
        if not k.coprime(p) {
            not_coprime_imp_divides_prime(p, k)
            p.divides(k)
        }
        if not k.coprime(q) {
            not_coprime_imp_divides_prime(q, k)
            q.divides(k)
        }
    }
}

/// Backward: if `p` or `q` divides `k` for distinct primes `p`, `q`, then `k`
/// is not coprime to `p * q`.
theorem div_imp_not_coprime_pq(p: Nat, q: Nat, k: Nat) {
    p.is_prime and q.is_prime and p != q and (p.divides(k) or q.divides(k))
        implies not k.coprime(p * q)
} by {
    if p.is_prime and q.is_prime and p != q and (p.divides(k) or q.divides(k)) {
        coprime_of_distinct_primes(p, q)
        p.coprime(q)
        coprime_mul_iff(k, p, q)
        if p.divides(k) {
            divides_prime_imp_not_coprime(p, k)
            not k.coprime(p)
            not (k.coprime(p) and k.coprime(q))
        }
        if q.divides(k) {
            divides_prime_imp_not_coprime(q, k)
            not k.coprime(q)
            not (k.coprime(p) and k.coprime(q))
        }
        not k.coprime(p * q)
    }
}

/// Recurrence: stepping `k` past a `not coprime` index adds one to the count.
theorem count_not_coprime_suc_yes(n: Nat, k: Nat) {
    not k.coprime(n)
        implies count_not_coprime_to(n, k.suc) = count_not_coprime_to(n, k) + Nat.1
}

/// Recurrence: stepping `k` past a coprime index leaves the count alone.
theorem count_not_coprime_suc_no(n: Nat, k: Nat) {
    k.coprime(n)
        implies count_not_coprime_to(n, k.suc) = count_not_coprime_to(n, k)
}

/// Inclusion-exclusion identity over `[0, k)`: for distinct primes `p`, `q`,
/// the not-coprime-to-`p*q` count combined with the multiples-of-`p*q` count
/// equals the sum of multiples of `p` and multiples of `q`.
define ie_pq_pred(p: Nat, q: Nat) -> (Nat -> Bool) {
    function(k: Nat) {
        count_not_coprime_to(p * q, k) + count_multiples(p * q, k) =
            count_multiples(p, k) + count_multiples(q, k)
    }
}

/// Case both: p | k and q | k.
theorem ie_pq_step_both(p: Nat, q: Nat, k: Nat) {
    p.is_prime and q.is_prime and p != q and ie_pq_pred(p, q)(k)
        and p.divides(k) and q.divides(k)
        implies ie_pq_pred(p, q)(k.suc)
} by {
    if p.is_prime and q.is_prime and p != q and ie_pq_pred(p, q)(k)
        and p.divides(k) and q.divides(k) {
        coprime_of_distinct_primes(p, q)
        let nc: Nat = count_not_coprime_to(p * q, k)
        let mpq: Nat = count_multiples(p * q, k)
        let mp: Nat = count_multiples(p, k)
        let mq: Nat = count_multiples(q, k)
        nc + mpq = mp + mq
        nat_coprime_combine(p, q, k)
        (p * q).divides(k)
        div_imp_not_coprime_pq(p, q, k)
        not k.coprime(p * q)
        count_not_coprime_suc_yes(p * q, k)
        count_not_coprime_to(p * q, k.suc) = nc + Nat.1
        count_multiples_suc_yes(p * q, k)
        count_multiples(p * q, k.suc) = mpq + Nat.1
        count_multiples_suc_yes(p, k)
        count_multiples(p, k.suc) = mp + Nat.1
        count_multiples_suc_yes(q, k)
        count_multiples(q, k.suc) = mq + Nat.1
        (nc + Nat.1) + (mpq + Nat.1) = nc + mpq + Nat.2
        (mp + Nat.1) + (mq + Nat.1) = mp + mq + Nat.2
        count_not_coprime_to(p * q, k.suc) + count_multiples(p * q, k.suc) =
            count_multiples(p, k.suc) + count_multiples(q, k.suc)
    }
}

/// Case left only: p | k and not q | k.
theorem ie_pq_step_left(p: Nat, q: Nat, k: Nat) {
    p.is_prime and q.is_prime and p != q and ie_pq_pred(p, q)(k)
        and p.divides(k) and not q.divides(k)
        implies ie_pq_pred(p, q)(k.suc)
} by {
    if p.is_prime and q.is_prime and p != q and ie_pq_pred(p, q)(k)
        and p.divides(k) and not q.divides(k) {
        let nc: Nat = count_not_coprime_to(p * q, k)
        let mpq: Nat = count_multiples(p * q, k)
        let mp: Nat = count_multiples(p, k)
        let mq: Nat = count_multiples(q, k)
        nc + mpq = mp + mq
        div_imp_not_coprime_pq(p, q, k)
        not k.coprime(p * q)
        count_not_coprime_suc_yes(p * q, k)
        count_not_coprime_to(p * q, k.suc) = nc + Nat.1
        if (p * q).divides(k) {
            divides_mul_imp_each(p, q, k)
            q.divides(k)
            false
        }
        not (p * q).divides(k)
        count_multiples_suc_no(p * q, k)
        count_multiples(p * q, k.suc) = mpq
        count_multiples_suc_yes(p, k)
        count_multiples(p, k.suc) = mp + Nat.1
        count_multiples_suc_no(q, k)
        count_multiples(q, k.suc) = mq
        (nc + Nat.1) + mpq = nc + mpq + Nat.1
        (mp + Nat.1) + mq = mp + mq + Nat.1
        count_not_coprime_to(p * q, k.suc) + count_multiples(p * q, k.suc) =
            count_multiples(p, k.suc) + count_multiples(q, k.suc)
    }
}

/// Case right only: not p | k and q | k.
theorem ie_pq_step_right(p: Nat, q: Nat, k: Nat) {
    p.is_prime and q.is_prime and p != q and ie_pq_pred(p, q)(k)
        and not p.divides(k) and q.divides(k)
        implies ie_pq_pred(p, q)(k.suc)
} by {
    if p.is_prime and q.is_prime and p != q and ie_pq_pred(p, q)(k)
        and not p.divides(k) and q.divides(k) {
        let nc: Nat = count_not_coprime_to(p * q, k)
        let mpq: Nat = count_multiples(p * q, k)
        let mp: Nat = count_multiples(p, k)
        let mq: Nat = count_multiples(q, k)
        nc + mpq = mp + mq
        div_imp_not_coprime_pq(p, q, k)
        not k.coprime(p * q)
        count_not_coprime_suc_yes(p * q, k)
        count_not_coprime_to(p * q, k.suc) = nc + Nat.1
        if (p * q).divides(k) {
            divides_mul_imp_each(p, q, k)
            p.divides(k)
            false
        }
        not (p * q).divides(k)
        count_multiples_suc_no(p * q, k)
        count_multiples(p * q, k.suc) = mpq
        count_multiples_suc_no(p, k)
        count_multiples(p, k.suc) = mp
        count_multiples_suc_yes(q, k)
        count_multiples(q, k.suc) = mq + Nat.1
        (nc + Nat.1) + mpq = nc + mpq + Nat.1
        mp + (mq + Nat.1) = mp + mq + Nat.1
        count_not_coprime_to(p * q, k.suc) + count_multiples(p * q, k.suc) =
            count_multiples(p, k.suc) + count_multiples(q, k.suc)
    }
}

/// Case neither: not p | k and not q | k (so k is coprime to p*q).
theorem ie_pq_step_neither(p: Nat, q: Nat, k: Nat) {
    p.is_prime and q.is_prime and p != q and ie_pq_pred(p, q)(k)
        and not p.divides(k) and not q.divides(k)
        implies ie_pq_pred(p, q)(k.suc)
} by {
    if p.is_prime and q.is_prime and p != q and ie_pq_pred(p, q)(k)
        and not p.divides(k) and not q.divides(k) {
        let nc: Nat = count_not_coprime_to(p * q, k)
        let mpq: Nat = count_multiples(p * q, k)
        let mp: Nat = count_multiples(p, k)
        let mq: Nat = count_multiples(q, k)
        nc + mpq = mp + mq
        if not k.coprime(p * q) {
            not_coprime_pq_imp_div(p, q, k)
            p.divides(k) or q.divides(k)
            false
        }
        k.coprime(p * q)
        count_not_coprime_suc_no(p * q, k)
        count_not_coprime_to(p * q, k.suc) = nc
        if (p * q).divides(k) {
            divides_mul_imp_each(p, q, k)
            p.divides(k)
            false
        }
        not (p * q).divides(k)
        count_multiples_suc_no(p * q, k)
        count_multiples(p * q, k.suc) = mpq
        count_multiples_suc_no(p, k)
        count_multiples(p, k.suc) = mp
        count_multiples_suc_no(q, k)
        count_multiples(q, k.suc) = mq
        count_not_coprime_to(p * q, k.suc) + count_multiples(p * q, k.suc) =
            count_multiples(p, k.suc) + count_multiples(q, k.suc)
    }
}

/// Inductive step for the inclusion-exclusion identity `ie_pq_pred`,
/// outer-on-p case split.
theorem ie_pq_step_p_div(p: Nat, q: Nat, k: Nat) {
    p.is_prime and q.is_prime and p != q and ie_pq_pred(p, q)(k)
        and p.divides(k)
        implies ie_pq_pred(p, q)(k.suc)
} by {
    if p.is_prime and q.is_prime and p != q and ie_pq_pred(p, q)(k)
        and p.divides(k) {
        if q.divides(k) {
            ie_pq_step_both(p, q, k)
        } else {
            ie_pq_step_left(p, q, k)
        }
    }
}

/// Inductive step for the inclusion-exclusion identity `ie_pq_pred`,
/// outer-on-not-p case split.
theorem ie_pq_step_not_p_div(p: Nat, q: Nat, k: Nat) {
    p.is_prime and q.is_prime and p != q and ie_pq_pred(p, q)(k)
        and not p.divides(k)
        implies ie_pq_pred(p, q)(k.suc)
} by {
    if p.is_prime and q.is_prime and p != q and ie_pq_pred(p, q)(k)
        and not p.divides(k) {
        if q.divides(k) {
            ie_pq_step_right(p, q, k)
        } else {
            ie_pq_step_neither(p, q, k)
        }
    }
}

/// Inductive step for the inclusion-exclusion identity `ie_pq_pred`.
theorem ie_pq_step(p: Nat, q: Nat, k: Nat) {
    p.is_prime and q.is_prime and p != q and ie_pq_pred(p, q)(k)
        implies ie_pq_pred(p, q)(k.suc)
} by {
    if p.is_prime and q.is_prime and p != q and ie_pq_pred(p, q)(k) {
        if p.divides(k) {
            ie_pq_step_p_div(p, q, k)
        } else {
            ie_pq_step_not_p_div(p, q, k)
        }
    }
}

/// Inner induction for the inclusion-exclusion identity: walks `k` upward.
theorem ie_pq_run(p: Nat, q: Nat, k: Nat) {
    p.is_prime and q.is_prime and p != q implies ie_pq_pred(p, q)(k)
} by {
    if p.is_prime and q.is_prime and p != q {
        let f: Nat -> Bool = function(x: Nat) {
            count_not_coprime_to(p * q, x) + count_multiples(p * q, x) =
                count_multiples(p, x) + count_multiples(q, x)
        }
        forall(y: Nat) {
            ie_pq_pred(p, q)(y) = f(y)
            f(y) = ie_pq_pred(p, q)(y)
        }
        // Base.
        count_not_coprime_to(p * q, Nat.0) = Nat.0
        count_multiples(p * q, Nat.0) = Nat.0
        count_multiples(p, Nat.0) = Nat.0
        count_multiples(q, Nat.0) = Nat.0
        f(Nat.0)
        // Step.
        forall(x: Nat) {
            if f(x) {
                ie_pq_pred(p, q)(x)
                ie_pq_step(p, q, x)
                ie_pq_pred(p, q)(x.suc)
                f(x.suc)
            }
        }
        f(k)
        ie_pq_pred(p, q)(k)
    }
}

/// `count_not_coprime_to(p * q, p * q) = p + q - 1` for distinct primes `p`, `q`.
theorem count_not_coprime_pq_at_pq(p: Nat, q: Nat) {
    p.is_prime and q.is_prime and p != q
        implies count_not_coprime_to(p * q, p * q) = p + q - Nat.1
} by {
    if p.is_prime and q.is_prime and p != q {
        ie_pq_run(p, q, p * q)
        count_not_coprime_to(p * q, p * q) + count_multiples(p * q, p * q) =
            count_multiples(p, p * q) + count_multiples(q, p * q)
        // count_multiples(p, p*q) = q.
        Nat.1 < p
        p != Nat.0
        count_multiples_div(p, q)
        count_multiples(p, p * q) = q
        // count_multiples(q, p*q) = p.
        Nat.1 < q
        q != Nat.0
        count_multiples_div(q, p)
        count_multiples(q, q * p) = p
        q * p = p * q
        count_multiples(q, p * q) = p
        // count_multiples(p*q, p*q) = 1.
        p * q != Nat.0
        p * q = (p * q) * Nat.1
        count_multiples_div(p * q, Nat.1)
        count_multiples(p * q, (p * q) * Nat.1) = Nat.1
        count_multiples(p * q, p * q) = Nat.1
        // Solve: count_not_coprime_to + 1 = q + p, so = p + q - 1.
        count_not_coprime_to(p * q, p * q) + Nat.1 = q + p
        q + p = p + q
        count_not_coprime_to(p * q, p * q) + Nat.1 = p + q
        Nat.1 <= p + q
        let pq_pred: Nat satisfy { pq_pred.suc = p + q }
        pq_pred + Nat.1 = pq_pred.suc
        pq_pred + Nat.1 = p + q
        count_not_coprime_to(p * q, p * q) = pq_pred
        // p + q - 1 = pq_pred.
        totient_sub_one_pred(p + q, pq_pred)
        (p + q) - Nat.1 = pq_pred
        count_not_coprime_to(p * q, p * q) = (p + q) - Nat.1
    }
}

/// Euler's totient at a product of distinct primes:
///   `nat_totient(p * q) = (p - 1) * (q - 1)`.
theorem totient_pq(p: Nat, q: Nat) {
    p.is_prime and q.is_prime and p != q
        implies (p * q).totient = (p - Nat.1) * (q - Nat.1)
} by {
    if p.is_prime and q.is_prime and p != q {
        coprime_partition(p * q, p * q)
        count_coprime_to(p * q, p * q) + count_not_coprime_to(p * q, p * q) = p * q
        count_not_coprime_pq_at_pq(p, q)
        count_not_coprime_to(p * q, p * q) = p + q - Nat.1
        count_coprime_to(p * q, p * q) + (p + q - Nat.1) = p * q
        // Solve for count_coprime_to: = p*q - (p + q - 1) = p*q - p - q + 1 = (p-1)(q-1).
        Nat.1 < p
        Nat.1 < q
        let pp: Nat satisfy { pp.suc = p }
        let qp: Nat satisfy { qp.suc = q }
        // (p - 1) = pp, (q - 1) = qp.
        totient_sub_one_pred(p, pp)
        p - Nat.1 = pp
        totient_sub_one_pred(q, qp)
        q - Nat.1 = qp
        // (p-1)*(q-1) = pp * qp.
        // p*q = (pp+1)*(qp+1) = pp*qp + pp + qp + 1.
        p = pp + Nat.1
        q = qp + Nat.1
        p * q = (pp + Nat.1) * (qp + Nat.1)
        (pp + Nat.1) * (qp + Nat.1) = pp * qp + pp + qp + Nat.1
        p * q = pp * qp + pp + qp + Nat.1
        // p + q - 1 = (pp + 1) + (qp + 1) - 1 = pp + qp + 1.
        p + q = (pp + Nat.1) + (qp + Nat.1)
        (pp + Nat.1) + (qp + Nat.1) = pp + qp + Nat.2
        p + q = pp + qp + Nat.2
        // p + q - 1 = pp + qp + 1.
        let s: Nat = pp + qp + Nat.1
        s + Nat.1 = pp + qp + Nat.2
        totient_sub_one_pred(p + q, s)
        p + q - Nat.1 = s
        // Now: count_coprime_to + (pp + qp + 1) = pp*qp + pp + qp + 1.
        count_coprime_to(p * q, p * q) + (pp + qp + Nat.1) = pp * qp + pp + qp + Nat.1
        // Rewrite RHS to (pp * qp) + (pp + qp + 1) and cancel via add_imp_sub.
        pp * qp + pp + qp + Nat.1 = pp * qp + (pp + qp + Nat.1)
        count_coprime_to(p * q, p * q) + (pp + qp + Nat.1) =
            pp * qp + (pp + qp + Nat.1)
        let total: Nat = pp * qp + (pp + qp + Nat.1)
        count_coprime_to(p * q, p * q) + (pp + qp + Nat.1) = total
        pp * qp + (pp + qp + Nat.1) = total
        add_imp_sub(count_coprime_to(p * q, p * q), pp + qp + Nat.1, total)
        add_imp_sub(pp * qp, pp + qp + Nat.1, total)
        count_coprime_to(p * q, p * q) = total - (pp + qp + Nat.1)
        pp * qp = total - (pp + qp + Nat.1)
        count_coprime_to(p * q, p * q) = pp * qp
        (p * q).totient = count_coprime_to(p * q, p * q)
        (p * q).totient = pp * qp
        (p - Nat.1) * (q - Nat.1) = pp * qp
        (p * q).totient = (p - Nat.1) * (q - Nat.1)
    }
}

/// For distinct primes `p`, `q` and any `m`,
///   `m^((p - 1) * (q - 1) + 1) ≡ m  (mod p * q)`.
/// Combines `rsa_pow_congr` mod `p` and mod `q` via the CRT.
theorem rsa_pow_congr_pq(p: Nat, q: Nat, m: Nat) {
    p.is_prime and q.is_prime and p != q
        implies m.pow((p - Nat.1) * (q - Nat.1) + Nat.1).congr_mod(m, p * q)
} by {
    if p.is_prime and q.is_prime and p != q {
        // Mod p: with k = q - 1, m^((q - 1) * (p - 1) + 1) ≡ m (mod p).
        rsa_pow_congr(p, m, q - Nat.1)
        m.pow((q - Nat.1) * (p - Nat.1) + Nat.1).congr_mod(m, p)
        (q - Nat.1) * (p - Nat.1) = (p - Nat.1) * (q - Nat.1)
        (q - Nat.1) * (p - Nat.1) + Nat.1 = (p - Nat.1) * (q - Nat.1) + Nat.1
        m.pow((p - Nat.1) * (q - Nat.1) + Nat.1).congr_mod(m, p)
        // Mod q: with k = p - 1, m^((p - 1) * (q - 1) + 1) ≡ m (mod q).
        rsa_pow_congr(q, m, p - Nat.1)
        m.pow((p - Nat.1) * (q - Nat.1) + Nat.1).congr_mod(m, q)
        // Combine via coprime CRT.
        coprime_of_distinct_primes(p, q)
        p.coprime(q)
        nat_congr_combine_coprime(p, q,
            m.pow((p - Nat.1) * (q - Nat.1) + Nat.1), m)
        m.pow((p - Nat.1) * (q - Nat.1) + Nat.1).congr_mod(m, p * q)
    }
}

/// Helper: m^((p-1)(q-1) + 1) = m * m^((p-1)(q-1)).
theorem rsa_pow_split_one(m: Nat, p: Nat, q: Nat) {
    m.pow((p - Nat.1) * (q - Nat.1) + Nat.1) =
        m * m.pow((p - Nat.1) * (q - Nat.1))
} by {
    let e: Nat = (p - Nat.1) * (q - Nat.1)
    exp_add(m, e, Nat.1)
    m.pow(e + Nat.1) = m.pow(e) * m.pow(Nat.1)
    exp_one(m)
    m.pow(Nat.1) = m
    m.pow(e + Nat.1) = m.pow(e) * m
    m.pow(e) * m = m * m.pow(e)
    m.pow(e + Nat.1) = m * m.pow(e)
}

/// Helper: rewrite `m * 1` as `m` and produce `m * m^e ≡ m * 1` from
/// `m * m^e ≡ m` after one extra rewrite.
theorem rsa_pow_one_rhs(m: Nat, n: Nat, e: Nat) {
    (m * m.pow(e)).congr_mod(m, n)
        implies (m * m.pow(e)).congr_mod(m * Nat.1, n)
} by {
    if (m * m.pow(e)).congr_mod(m, n) {
        m * Nat.1 = m
    }
}

/// Euler's theorem at a product of distinct primes:
///   `gcd(m, p * q) = 1` implies `m.pow((p * q).totient) ≡ 1  (mod p * q)`.
theorem euler_pq(p: Nat, q: Nat, m: Nat) {
    p.is_prime and q.is_prime and p != q and m.coprime(p * q)
        implies m.pow((p * q).totient).congr_mod(Nat.1, p * q)
} by {
    if p.is_prime and q.is_prime and p != q and m.coprime(p * q) {
        // Identify totient(p*q) with (p-1)*(q-1).
        totient_pq(p, q)
        (p * q).totient = (p - Nat.1) * (q - Nat.1)
        // From rsa_pow_congr_pq: m^((p-1)(q-1) + 1) ≡ m (mod p*q).
        rsa_pow_congr_pq(p, q, m)
        m.pow((p - Nat.1) * (q - Nat.1) + Nat.1).congr_mod(m, p * q)
        // Rewrite LHS as m * m^((p-1)(q-1)).
        rsa_pow_split_one(m, p, q)
        m.pow((p - Nat.1) * (q - Nat.1) + Nat.1) =
            m * m.pow((p - Nat.1) * (q - Nat.1))
        (m * m.pow((p - Nat.1) * (q - Nat.1))).congr_mod(m, p * q)
        // Rewrite RHS as m * 1.
        rsa_pow_one_rhs(m, p * q, (p - Nat.1) * (q - Nat.1))
        (m * m.pow((p - Nat.1) * (q - Nat.1))).congr_mod(m * Nat.1, p * q)
        // Cancel m using its coprimality.
        cancel_coprime(m, p * q, m.pow((p - Nat.1) * (q - Nat.1)), Nat.1)
        m.pow((p - Nat.1) * (q - Nat.1)).congr_mod(Nat.1, p * q)
        m.pow((p * q).totient).congr_mod(Nat.1, p * q)
    }
}

/// Forward: for prime `p`, "not coprime to `p * p`" implies `p` divides.
theorem not_coprime_pp_imp_divides(p: Nat, k: Nat) {
    p.is_prime and not k.coprime(p * p) implies p.divides(k)
} by {
    if p.is_prime and not k.coprime(p * p) {
        coprime_mul_iff(k, p, p)
        not (k.coprime(p) and k.coprime(p))
        not k.coprime(p)
        not_coprime_imp_divides_prime(p, k)
    }
}

/// Backward: for prime `p`, "p divides k" implies "not coprime to p * p".
theorem divides_imp_not_coprime_pp(p: Nat, k: Nat) {
    p.is_prime and p.divides(k) implies not k.coprime(p * p)
} by {
    if p.is_prime and p.divides(k) {
        coprime_mul_iff(k, p, p)
        divides_prime_imp_not_coprime(p, k)
        not k.coprime(p)
        not (k.coprime(p) and k.coprime(p))
    }
}

/// Inductive predicate for the prime-square count equivalence.
define pp_count_pred(p: Nat) -> (Nat -> Bool) {
    function(k: Nat) {
        count_not_coprime_to(p * p, k) = count_multiples(p, k)
    }
}

/// Inductive step: for prime `p`, `count_not_coprime_to(p * p, _)` and
/// `count_multiples(p, _)` advance in lockstep.
theorem pp_count_step(p: Nat, k: Nat) {
    p.is_prime and pp_count_pred(p)(k)
        implies pp_count_pred(p)(k.suc)
} by {
    if p.is_prime and pp_count_pred(p)(k) {
        count_not_coprime_to(p * p, k) = count_multiples(p, k)
        if p.divides(k) {
            divides_imp_not_coprime_pp(p, k)
            not k.coprime(p * p)
            count_not_coprime_suc_yes(p * p, k)
            count_not_coprime_to(p * p, k.suc) = count_not_coprime_to(p * p, k) + Nat.1
            count_multiples_suc_yes(p, k)
            count_multiples(p, k.suc) = count_multiples(p, k) + Nat.1
            count_not_coprime_to(p * p, k.suc) = count_multiples(p, k.suc)
        } else {
            not p.divides(k)
            if not k.coprime(p * p) {
                not_coprime_pp_imp_divides(p, k)
                p.divides(k)
                false
            }
            k.coprime(p * p)
            count_not_coprime_suc_no(p * p, k)
            count_not_coprime_to(p * p, k.suc) = count_not_coprime_to(p * p, k)
            count_multiples_suc_no(p, k)
            count_multiples(p, k.suc) = count_multiples(p, k)
            count_not_coprime_to(p * p, k.suc) = count_multiples(p, k.suc)
        }
    }
}

/// Inner induction for the prime-square count equivalence.
theorem pp_count_run(p: Nat, k: Nat) {
    p.is_prime implies pp_count_pred(p)(k)
} by {
    if p.is_prime {
        let f: Nat -> Bool = function(x: Nat) {
            count_not_coprime_to(p * p, x) = count_multiples(p, x)
        }
        forall(y: Nat) {
            pp_count_pred(p)(y) = f(y)
            f(y) = pp_count_pred(p)(y)
        }
        // Base.
        count_not_coprime_to(p * p, Nat.0) = Nat.0
        count_multiples(p, Nat.0) = Nat.0
        f(Nat.0)
        // Step.
        forall(x: Nat) {
            if f(x) {
                pp_count_pred(p)(x)
                pp_count_step(p, x)
                pp_count_pred(p)(x.suc)
                f(x.suc)
            }
        }
        f(k)
        pp_count_pred(p)(k)
    }
}

/// `count_not_coprime_to(p * p, k) = count_multiples(p, k)` for prime `p`.
theorem pp_count_eq(p: Nat, k: Nat) {
    p.is_prime
        implies count_not_coprime_to(p * p, k) = count_multiples(p, k)
} by {
    if p.is_prime {
        pp_count_run(p, k)
        pp_count_pred(p)(k)
    }
}

/// Euler's totient at the square of a prime: `totient(p * p) = p * p - p`.
theorem totient_pp(p: Nat) {
    p.is_prime implies (p * p).totient = p * p - p
} by {
    if p.is_prime {
        Nat.1 < p
        p != Nat.0
        // count_not_coprime_to(p * p, p * p) = count_multiples(p, p * p) = p.
        pp_count_eq(p, p * p)
        count_not_coprime_to(p * p, p * p) = count_multiples(p, p * p)
        count_multiples_div(p, p)
        count_multiples(p, p * p) = p
        count_not_coprime_to(p * p, p * p) = p
        // count_coprime_to(p * p, p * p) + p = p * p, so = p * p - p.
        coprime_partition(p * p, p * p)
        count_coprime_to(p * p, p * p) + count_not_coprime_to(p * p, p * p) = p * p
        count_coprime_to(p * p, p * p) + p = p * p
        // Solve via add_imp_sub.
        add_imp_sub(count_coprime_to(p * p, p * p), p, p * p)
        count_coprime_to(p * p, p * p) = p * p - p
        (p * p).totient = count_coprime_to(p * p, p * p)
        (p * p).totient = p * p - p
    }
}

/// Inductive predicate for `coprime_pow_iff`: at each `n`, coprimality with
/// `p^(n.suc)` agrees with coprimality with `p`.
define coprime_pow_pred(k: Nat, p: Nat) -> (Nat -> Bool) {
    function(n: Nat) {
        k.coprime(p.pow(n.suc)) = k.coprime(p)
    }
}

/// Base case: `k.coprime(p^1) = k.coprime(p)` since `p^1 = p`.
theorem coprime_pow_base(k: Nat, p: Nat) {
    coprime_pow_pred(k, p)(Nat.0)
} by {
    exp_one(p)
    p.pow(Nat.1) = p
    Nat.0.suc = Nat.1
    p.pow(Nat.0.suc) = p
    k.coprime(p.pow(Nat.0.suc)) = k.coprime(p)
}

/// Inductive step: from `k.coprime(p^n.suc) = k.coprime(p)`, deduce the same
/// for `n.suc.suc`.
theorem coprime_pow_step(k: Nat, p: Nat, n: Nat) {
    coprime_pow_pred(k, p)(n) implies coprime_pow_pred(k, p)(n.suc)
} by {
    if coprime_pow_pred(k, p)(n) {
        k.coprime(p.pow(n.suc)) = k.coprime(p)
        // p^(n.suc.suc) = p * p^(n.suc).
        exp_add(p, Nat.1, n.suc)
        Nat.1 + n.suc = n.suc.suc
        p.pow(Nat.1 + n.suc) = p.pow(Nat.1) * p.pow(n.suc)
        p.pow(n.suc.suc) = p.pow(Nat.1) * p.pow(n.suc)
        exp_one(p)
        p.pow(Nat.1) = p
        p.pow(n.suc.suc) = p * p.pow(n.suc)
        // Coprime split.
        coprime_mul_iff(k, p, p.pow(n.suc))
        if k.coprime(p * p.pow(n.suc)) {
            k.coprime(p) and k.coprime(p.pow(n.suc))
            k.coprime(p)
        }
        if k.coprime(p) {
            k.coprime(p.pow(n.suc))
            k.coprime(p) and k.coprime(p.pow(n.suc))
            k.coprime(p * p.pow(n.suc))
        }
        k.coprime(p * p.pow(n.suc)) = k.coprime(p)
        k.coprime(p.pow(n.suc.suc)) = k.coprime(p)
    }
}

/// For prime `p` and `n >= 1`, `k.coprime(p^n) = k.coprime(p)`. Stated using
/// `n.suc` so the precondition `n >= 1` is automatic.
theorem coprime_pow_iff(k: Nat, p: Nat, n: Nat) {
    k.coprime(p.pow(n.suc)) = k.coprime(p)
} by {
    let f: Nat -> Bool = function(x: Nat) {
        k.coprime(p.pow(x.suc)) = k.coprime(p)
    }
    forall(y: Nat) {
        coprime_pow_pred(k, p)(y) = f(y)
        f(y) = coprime_pow_pred(k, p)(y)
    }
    coprime_pow_base(k, p)
    coprime_pow_pred(k, p)(Nat.0)
    f(Nat.0)
    forall(x: Nat) {
        if f(x) {
            coprime_pow_pred(k, p)(x)
            coprime_pow_step(k, p, x)
            coprime_pow_pred(k, p)(x.suc)
            f(x.suc)
        }
    }
    f(n)
}

/// For prime `p` and `n >= 1`: "not coprime to `p^n`" iff `p` divides.
theorem not_coprime_pow_iff_divides_prime(p: Nat, n: Nat, k: Nat) {
    p.is_prime
        implies (not k.coprime(p.pow(n.suc)) = p.divides(k))
} by {
    if p.is_prime {
        coprime_pow_iff(k, p, n)
        // k.coprime(p^(n.suc)) = k.coprime(p).
        not_coprime_iff_divides_prime(p, k)
        // not k.coprime(p) = p.divides(k).
        if not k.coprime(p.pow(n.suc)) {
            not k.coprime(p)
            p.divides(k)
        }
        if p.divides(k) {
            not k.coprime(p)
            not k.coprime(p.pow(n.suc))
        }
    }
}

/// Inductive predicate for the prime-power count equivalence.
define ppow_count_pred(p: Nat, n: Nat) -> (Nat -> Bool) {
    function(k: Nat) {
        count_not_coprime_to(p.pow(n.suc), k) = count_multiples(p, k)
    }
}

/// Inductive step.
theorem ppow_count_step(p: Nat, n: Nat, k: Nat) {
    p.is_prime and ppow_count_pred(p, n)(k)
        implies ppow_count_pred(p, n)(k.suc)
} by {
    if p.is_prime and ppow_count_pred(p, n)(k) {
        count_not_coprime_to(p.pow(n.suc), k) = count_multiples(p, k)
        not_coprime_pow_iff_divides_prime(p, n, k)
        if p.divides(k) {
            not k.coprime(p.pow(n.suc))
            count_not_coprime_suc_yes(p.pow(n.suc), k)
            count_not_coprime_to(p.pow(n.suc), k.suc) =
                count_not_coprime_to(p.pow(n.suc), k) + Nat.1
            count_multiples_suc_yes(p, k)
            count_multiples(p, k.suc) = count_multiples(p, k) + Nat.1
            count_not_coprime_to(p.pow(n.suc), k.suc) = count_multiples(p, k.suc)
        } else {
            not p.divides(k)
            if not k.coprime(p.pow(n.suc)) {
                p.divides(k)
                false
            }
            k.coprime(p.pow(n.suc))
            count_not_coprime_suc_no(p.pow(n.suc), k)
            count_not_coprime_to(p.pow(n.suc), k.suc) =
                count_not_coprime_to(p.pow(n.suc), k)
            count_multiples_suc_no(p, k)
            count_multiples(p, k.suc) = count_multiples(p, k)
            count_not_coprime_to(p.pow(n.suc), k.suc) = count_multiples(p, k.suc)
        }
    }
}

/// Inner induction for the prime-power count equivalence.
theorem ppow_count_run(p: Nat, n: Nat, k: Nat) {
    p.is_prime implies ppow_count_pred(p, n)(k)
} by {
    if p.is_prime {
        let f: Nat -> Bool = function(x: Nat) {
            count_not_coprime_to(p.pow(n.suc), x) = count_multiples(p, x)
        }
        forall(y: Nat) {
            ppow_count_pred(p, n)(y) = f(y)
            f(y) = ppow_count_pred(p, n)(y)
        }
        count_not_coprime_to(p.pow(n.suc), Nat.0) = Nat.0
        count_multiples(p, Nat.0) = Nat.0
        f(Nat.0)
        forall(x: Nat) {
            if f(x) {
                ppow_count_pred(p, n)(x)
                ppow_count_step(p, n, x)
                ppow_count_pred(p, n)(x.suc)
                f(x.suc)
            }
        }
        f(k)
        ppow_count_pred(p, n)(k)
    }
}

/// Helper: `p^(n.suc) = p * p^n`, exposing the leading factor for
/// `count_multiples_div`.
theorem ppow_split(p: Nat, n: Nat) {
    p.pow(n.suc) = p * p.pow(n)
} by {
    exp_add(p, Nat.1, n)
    Nat.1 + n = n.suc
    p.pow(Nat.1 + n) = p.pow(Nat.1) * p.pow(n)
    p.pow(n.suc) = p.pow(Nat.1) * p.pow(n)
    exp_one(p)
    p.pow(Nat.1) = p
    p.pow(n.suc) = p * p.pow(n)
}

/// Euler's totient at a prime power:
///   `totient(p^(n.suc)) = p^(n.suc) - p^n` for prime `p`.
theorem totient_p_pow(p: Nat, n: Nat) {
    p.is_prime implies (p.pow(n.suc)).totient = p.pow(n.suc) - p.pow(n)
} by {
    if p.is_prime {
        Nat.1 < p
        p != Nat.0
        ppow_count_run(p, n, p.pow(n.suc))
        ppow_count_pred(p, n)(p.pow(n.suc))
        count_not_coprime_to(p.pow(n.suc), p.pow(n.suc)) =
            count_multiples(p, p.pow(n.suc))
        // count_multiples(p, p * p^n) = p^n.
        ppow_split(p, n)
        p.pow(n.suc) = p * p.pow(n)
        count_multiples_div(p, p.pow(n))
        count_multiples(p, p * p.pow(n)) = p.pow(n)
        count_multiples(p, p.pow(n.suc)) = p.pow(n)
        count_not_coprime_to(p.pow(n.suc), p.pow(n.suc)) = p.pow(n)
        // Apply coprime_partition.
        coprime_partition(p.pow(n.suc), p.pow(n.suc))
        count_coprime_to(p.pow(n.suc), p.pow(n.suc)) +
            count_not_coprime_to(p.pow(n.suc), p.pow(n.suc)) = p.pow(n.suc)
        count_coprime_to(p.pow(n.suc), p.pow(n.suc)) + p.pow(n) = p.pow(n.suc)
        add_imp_sub(count_coprime_to(p.pow(n.suc), p.pow(n.suc)),
            p.pow(n), p.pow(n.suc))
        count_coprime_to(p.pow(n.suc), p.pow(n.suc)) = p.pow(n.suc) - p.pow(n)
        (p.pow(n.suc)).totient = count_coprime_to(p.pow(n.suc), p.pow(n.suc))
        (p.pow(n.suc)).totient = p.pow(n.suc) - p.pow(n)
    }
}

/// Helper: `p * x - x = (p - 1) * x` for `p >= 1`.
theorem mul_sub_one_factor(p: Nat, x: Nat) {
    Nat.1 <= p implies p * x - x = (p - Nat.1) * x
} by {
    if Nat.1 <= p {
        Nat.0 < p
        p != Nat.0
        let pp: Nat satisfy { pp.suc = p }
        totient_sub_one_pred(p, pp)
        p - Nat.1 = pp
        p * x = pp.suc * x
        pp.suc * x = (pp + Nat.1) * x
        (pp + Nat.1) * x = pp * x + Nat.1 * x
        Nat.1 * x = x
        pp * x + Nat.1 * x = pp * x + x
        p * x = pp * x + x
        add_imp_sub(pp * x, x, p * x)
        p * x - x = pp * x
        (p - Nat.1) * x = pp * x
        p * x - x = (p - Nat.1) * x
    }
}

/// Factored form of the prime-power totient:
///   `totient(p^(n.suc)) = (p - 1) * p^n` for prime `p`.
theorem totient_p_pow_factored(p: Nat, n: Nat) {
    p.is_prime implies (p.pow(n.suc)).totient = (p - Nat.1) * p.pow(n)
} by {
    if p.is_prime {
        Nat.1 < p
        Nat.1 <= p
        totient_p_pow(p, n)
        (p.pow(n.suc)).totient = p.pow(n.suc) - p.pow(n)
        ppow_split(p, n)
        p.pow(n.suc) = p * p.pow(n)
        (p.pow(n.suc)).totient = p * p.pow(n) - p.pow(n)
        mul_sub_one_factor(p, p.pow(n))
        p * p.pow(n) - p.pow(n) = (p - Nat.1) * p.pow(n)
        (p.pow(n.suc)).totient = (p - Nat.1) * p.pow(n)
    }
}

/// The list of natural numbers in `[0, k)` that are coprime to `n`, in
/// descending order. Mirror of `count_coprime_to` shaped to allow product /
/// permutation arguments downstream (e.g. for general Euler).
define coprime_residues_below(n: Nat, k: Nat) -> List[Nat] {
    match k {
        Nat.zero {
            List.nil[Nat]
        }
        Nat.suc(j) {
            if j.coprime(n) {
                List.cons(j, coprime_residues_below(n, j))
            } else {
                coprime_residues_below(n, j)
            }
        }
    }
}

/// The full coprime-residue list for modulus `n`: integers in `[0, n)`
/// coprime to `n`. Has length `nat_totient(n)`.
define coprime_residues(n: Nat) -> List[Nat] {
    coprime_residues_below(n, n)
}

/// Recurrence: stepping past a coprime index conses it onto the residue list.
theorem coprime_residues_below_suc_yes(n: Nat, k: Nat) {
    k.coprime(n)
        implies coprime_residues_below(n, k.suc) =
            List.cons(k, coprime_residues_below(n, k))
}

/// Recurrence: stepping past a non-coprime index leaves the residue list alone.
theorem coprime_residues_below_suc_no(n: Nat, k: Nat) {
    not k.coprime(n)
        implies coprime_residues_below(n, k.suc) =
            coprime_residues_below(n, k)
}

/// Inductive predicate for `coprime_residues_below_length`.
define coprime_residues_length_pred(n: Nat) -> (Nat -> Bool) {
    function(k: Nat) {
        coprime_residues_below(n, k).length = count_coprime_to(n, k)
    }
}

/// Inductive step: the list length and the count grow together at each k.
theorem coprime_residues_length_step(n: Nat, k: Nat) {
    coprime_residues_length_pred(n)(k)
        implies coprime_residues_length_pred(n)(k.suc)
} by {
    if coprime_residues_length_pred(n)(k) {
        coprime_residues_below(n, k).length = count_coprime_to(n, k)
        if k.coprime(n) {
            coprime_residues_below_suc_yes(n, k)
            coprime_residues_below(n, k.suc) =
                List.cons(k, coprime_residues_below(n, k))
            coprime_residues_below(n, k.suc).length =
                coprime_residues_below(n, k).length.suc
            count_coprime_to_suc_yes(n, k)
            count_coprime_to(n, k.suc) = count_coprime_to(n, k) + Nat.1
            count_coprime_to(n, k) + Nat.1 = count_coprime_to(n, k).suc
            coprime_residues_below(n, k.suc).length = count_coprime_to(n, k.suc)
        } else {
            coprime_residues_below_suc_no(n, k)
            coprime_residues_below(n, k.suc) = coprime_residues_below(n, k)
            count_coprime_to_suc_no(n, k)
            count_coprime_to(n, k.suc) = count_coprime_to(n, k)
            coprime_residues_below(n, k.suc).length = count_coprime_to(n, k.suc)
        }
    }
}

/// `coprime_residues_below(n, k).length = count_coprime_to(n, k)` by induction.
theorem coprime_residues_below_length(n: Nat, k: Nat) {
    coprime_residues_below(n, k).length = count_coprime_to(n, k)
} by {
    let f: Nat -> Bool = function(x: Nat) {
        coprime_residues_below(n, x).length = count_coprime_to(n, x)
    }
    forall(y: Nat) {
        coprime_residues_length_pred(n)(y) = f(y)
        f(y) = coprime_residues_length_pred(n)(y)
    }
    coprime_residues_below(n, Nat.0).length = Nat.0
    count_coprime_to(n, Nat.0) = Nat.0
    f(Nat.0)
    forall(x: Nat) {
        if f(x) {
            coprime_residues_length_pred(n)(x)
            coprime_residues_length_step(n, x)
            coprime_residues_length_pred(n)(x.suc)
            f(x.suc)
        }
    }
    f(k)
}

/// `coprime_residues(n).length = nat_totient(n)`.
theorem coprime_residues_length(n: Nat) {
    coprime_residues(n).length = n.totient
} by {
    coprime_residues_below_length(n, n)
    coprime_residues_below(n, n).length = count_coprime_to(n, n)
    coprime_residues(n).length = coprime_residues_below(n, n).length
    coprime_residues(n).length = count_coprime_to(n, n)
    n.totient = count_coprime_to(n, n)
    coprime_residues(n).length = n.totient
}

/// Inductive predicate for `coprime_residues_below_all_coprime`.
define cr_all_coprime_pred(n: Nat) -> (Nat -> Bool) {
    function(k: Nat) {
        forall(x: Nat) {
            coprime_residues_below(n, k).contains(x) implies x.coprime(n)
        }
    }
}

/// Bridge: cr_all_coprime_pred unfolds to its underlying forall.
theorem cr_all_coprime_pred_apply(n: Nat, k: Nat, x: Nat) {
    cr_all_coprime_pred(n)(k) and coprime_residues_below(n, k).contains(x)
        implies x.coprime(n)
} by {
    if cr_all_coprime_pred(n)(k) and coprime_residues_below(n, k).contains(x) {
        cr_all_coprime_pred(n)(k) = forall(y: Nat) {
            coprime_residues_below(n, k).contains(y) implies y.coprime(n)
        }
        forall(y: Nat) {
            coprime_residues_below(n, k).contains(y) implies y.coprime(n)
        }
    }
}

/// Inductive step.
theorem cr_all_coprime_step(n: Nat, k: Nat) {
    cr_all_coprime_pred(n)(k) implies cr_all_coprime_pred(n)(k.suc)
} by {
    if cr_all_coprime_pred(n)(k) {
        forall(x: Nat) {
            if coprime_residues_below(n, k.suc).contains(x) {
                if k.coprime(n) {
                    coprime_residues_below_suc_yes(n, k)
                    coprime_residues_below(n, k.suc) =
                        List.cons(k, coprime_residues_below(n, k))
                    List.cons(k, coprime_residues_below(n, k)).contains(x)
                    if x = k {
                        x.coprime(n)
                    } else {
                        x != k
                        coprime_residues_below(n, k).contains(x)
                        cr_all_coprime_pred_apply(n, k, x)
                        x.coprime(n)
                    }
                } else {
                    coprime_residues_below_suc_no(n, k)
                    coprime_residues_below(n, k.suc) = coprime_residues_below(n, k)
                    coprime_residues_below(n, k).contains(x)
                    cr_all_coprime_pred_apply(n, k, x)
                    x.coprime(n)
                }
            }
        }
    }
}

/// Every element of `coprime_residues_below(n, k)` is coprime to `n`.
theorem coprime_residues_below_all_coprime(n: Nat, k: Nat, x: Nat) {
    coprime_residues_below(n, k).contains(x) implies x.coprime(n)
} by {
    let f: Nat -> Bool = function(m: Nat) {
        forall(y: Nat) {
            coprime_residues_below(n, m).contains(y) implies y.coprime(n)
        }
    }
    forall(y: Nat) {
        cr_all_coprime_pred(n)(y) = f(y)
        f(y) = cr_all_coprime_pred(n)(y)
    }
    // Base.
    forall(y: Nat) {
        if coprime_residues_below(n, Nat.0).contains(y) {
            coprime_residues_below(n, Nat.0) = List.nil[Nat]
            List.nil[Nat].contains(y)
            false
        }
    }
    f(Nat.0)
    forall(m: Nat) {
        if f(m) {
            cr_all_coprime_pred(n)(m)
            cr_all_coprime_step(n, m)
            cr_all_coprime_pred(n)(m.suc)
            f(m.suc)
        }
    }
    f(k)
}

/// Every element of `coprime_residues(n)` is coprime to `n`.
theorem coprime_residues_all_coprime(n: Nat, x: Nat) {
    coprime_residues(n).contains(x) implies x.coprime(n)
} by {
    if coprime_residues(n).contains(x) {
        coprime_residues(n) = coprime_residues_below(n, n)
        coprime_residues_below(n, n).contains(x)
        coprime_residues_below_all_coprime(n, n, x)
    }
}

/// Inductive predicate for `coprime_residues_below_all_below`.
define cr_all_below_pred(n: Nat) -> (Nat -> Bool) {
    function(k: Nat) {
        forall(x: Nat) {
            coprime_residues_below(n, k).contains(x) implies x < k
        }
    }
}

/// Bridge: cr_all_below_pred unfolds to its underlying forall.
theorem cr_all_below_pred_apply(n: Nat, k: Nat, x: Nat) {
    cr_all_below_pred(n)(k) and coprime_residues_below(n, k).contains(x)
        implies x < k
} by {
    if cr_all_below_pred(n)(k) and coprime_residues_below(n, k).contains(x) {
        cr_all_below_pred(n)(k) = forall(y: Nat) {
            coprime_residues_below(n, k).contains(y) implies y < k
        }
        forall(y: Nat) {
            coprime_residues_below(n, k).contains(y) implies y < k
        }
    }
}

/// Inductive step: contained values are below the cap, recursively.
theorem cr_all_below_step(n: Nat, k: Nat) {
    cr_all_below_pred(n)(k) implies cr_all_below_pred(n)(k.suc)
} by {
    if cr_all_below_pred(n)(k) {
        forall(x: Nat) {
            if coprime_residues_below(n, k.suc).contains(x) {
                if k.coprime(n) {
                    coprime_residues_below_suc_yes(n, k)
                    coprime_residues_below(n, k.suc) =
                        List.cons(k, coprime_residues_below(n, k))
                    List.cons(k, coprime_residues_below(n, k)).contains(x)
                    if x = k {
                        k < k.suc
                        x < k.suc
                    } else {
                        x != k
                        coprime_residues_below(n, k).contains(x)
                        cr_all_below_pred_apply(n, k, x)
                        x < k
                        k < k.suc
                        x < k.suc
                    }
                } else {
                    coprime_residues_below_suc_no(n, k)
                    coprime_residues_below(n, k.suc) = coprime_residues_below(n, k)
                    coprime_residues_below(n, k).contains(x)
                    cr_all_below_pred_apply(n, k, x)
                    x < k
                    k < k.suc
                    x < k.suc
                }
            }
        }
    }
}

/// Every element of `coprime_residues_below(n, k)` is `< k`.
theorem coprime_residues_below_all_below(n: Nat, k: Nat, x: Nat) {
    coprime_residues_below(n, k).contains(x) implies x < k
} by {
    let f: Nat -> Bool = function(m: Nat) {
        forall(y: Nat) {
            coprime_residues_below(n, m).contains(y) implies y < m
        }
    }
    forall(y: Nat) {
        cr_all_below_pred(n)(y) = f(y)
        f(y) = cr_all_below_pred(n)(y)
    }
    // Base.
    forall(y: Nat) {
        if coprime_residues_below(n, Nat.0).contains(y) {
            coprime_residues_below(n, Nat.0) = List.nil[Nat]
            List.nil[Nat].contains(y)
            false
        }
    }
    f(Nat.0)
    forall(m: Nat) {
        if f(m) {
            cr_all_below_pred(n)(m)
            cr_all_below_step(n, m)
            cr_all_below_pred(n)(m.suc)
            f(m.suc)
        }
    }
    f(k)
}

/// Every element of `coprime_residues(n)` is `< n`.
theorem coprime_residues_all_below(n: Nat, x: Nat) {
    coprime_residues(n).contains(x) implies x < n
} by {
    if coprime_residues(n).contains(x) {
        coprime_residues(n) = coprime_residues_below(n, n)
        coprime_residues_below(n, n).contains(x)
        coprime_residues_below_all_below(n, n, x)
    }
}

/// Inductive predicate for `coprime_residues_below_contains`: every coprime
/// `x < k` lies in the residue list.
define cr_contains_pred(n: Nat) -> (Nat -> Bool) {
    function(k: Nat) {
        forall(x: Nat) {
            x < k and x.coprime(n) implies coprime_residues_below(n, k).contains(x)
        }
    }
}

/// Bridge: cr_contains_pred unfolds to its underlying forall.
theorem cr_contains_pred_apply(n: Nat, k: Nat, x: Nat) {
    cr_contains_pred(n)(k) and x < k and x.coprime(n)
        implies coprime_residues_below(n, k).contains(x)
} by {
    if cr_contains_pred(n)(k) and x < k and x.coprime(n) {
        cr_contains_pred(n)(k) = forall(y: Nat) {
            y < k and y.coprime(n) implies coprime_residues_below(n, k).contains(y)
        }
        forall(y: Nat) {
            y < k and y.coprime(n) implies coprime_residues_below(n, k).contains(y)
        }
    }
}

/// Inductive step.
theorem cr_contains_step(n: Nat, k: Nat) {
    cr_contains_pred(n)(k) implies cr_contains_pred(n)(k.suc)
} by {
    if cr_contains_pred(n)(k) {
        forall(x: Nat) {
            if x < k.suc and x.coprime(n) {
                if x = k {
                    // x = k, so k.coprime(n). Use yes-branch.
                    k.coprime(n)
                    coprime_residues_below_suc_yes(n, k)
                    coprime_residues_below(n, k.suc) =
                        List.cons(k, coprime_residues_below(n, k))
                    List.cons(k, coprime_residues_below(n, k)).contains(k)
                    coprime_residues_below(n, k.suc).contains(k)
                    coprime_residues_below(n, k.suc).contains(x)
                } else {
                    x != k
                    x < k
                    cr_contains_pred_apply(n, k, x)
                    coprime_residues_below(n, k).contains(x)
                    if k.coprime(n) {
                        coprime_residues_below_suc_yes(n, k)
                        coprime_residues_below(n, k.suc) =
                            List.cons(k, coprime_residues_below(n, k))
                        List.cons(k, coprime_residues_below(n, k)).contains(x)
                        coprime_residues_below(n, k.suc).contains(x)
                    } else {
                        coprime_residues_below_suc_no(n, k)
                        coprime_residues_below(n, k.suc) = coprime_residues_below(n, k)
                        coprime_residues_below(n, k.suc).contains(x)
                    }
                }
            }
        }
    }
}

/// Every coprime `x < k` lies in `coprime_residues_below(n, k)`.
theorem coprime_residues_below_contains(n: Nat, k: Nat, x: Nat) {
    x < k and x.coprime(n)
        implies coprime_residues_below(n, k).contains(x)
} by {
    let f: Nat -> Bool = function(m: Nat) {
        forall(y: Nat) {
            y < m and y.coprime(n) implies coprime_residues_below(n, m).contains(y)
        }
    }
    forall(y: Nat) {
        cr_contains_pred(n)(y) = f(y)
        f(y) = cr_contains_pred(n)(y)
    }
    // Base.
    forall(y: Nat) {
        if y < Nat.0 and y.coprime(n) {
            false
        }
    }
    f(Nat.0)
    forall(m: Nat) {
        if f(m) {
            cr_contains_pred(n)(m)
            cr_contains_step(n, m)
            cr_contains_pred(n)(m.suc)
            f(m.suc)
        }
    }
    f(k)
}

/// Forward: contained `x` lies in `[0, n)` and is coprime to `n`.
theorem coprime_residues_contains_imp(n: Nat, x: Nat) {
    coprime_residues(n).contains(x) implies x < n and x.coprime(n)
} by {
    if coprime_residues(n).contains(x) {
        coprime_residues_all_below(n, x)
        coprime_residues_all_coprime(n, x)
    }
}

/// Backward: every coprime `x < n` is in `coprime_residues(n)`.
theorem coprime_residues_contains_intro(n: Nat, x: Nat) {
    x < n and x.coprime(n) implies coprime_residues(n).contains(x)
} by {
    if x < n and x.coprime(n) {
        coprime_residues_below_contains(n, n, x)
        coprime_residues_below(n, n).contains(x)
        coprime_residues(n) = coprime_residues_below(n, n)
    }
}

/// Membership characterization for `coprime_residues(n)`: `x` is contained iff
/// `x < n` and `x` is coprime to `n`.
theorem coprime_residues_contains_iff(n: Nat, x: Nat) {
    coprime_residues(n).contains(x) = (x < n and x.coprime(n))
} by {
    if coprime_residues(n).contains(x) {
        coprime_residues_contains_imp(n, x)
    }
    if x < n and x.coprime(n) {
        coprime_residues_contains_intro(n, x)
        coprime_residues(n).contains(x)
    }
}

/// Helper: for `a` and `x` both coprime to `n`, the product is coprime to `n`.
theorem coprime_mul_both(n: Nat, a: Nat, x: Nat) {
    a.coprime(n) and x.coprime(n) implies (a * x).coprime(n)
} by {
    if a.coprime(n) and x.coprime(n) {
        coprime_comm(a, n)
        n.coprime(a)
        coprime_comm(x, n)
        n.coprime(x)
        coprime_mul(n, a, x)
        n.coprime(a * x)
        coprime_comm(n, a * x)
        (a * x).coprime(n)
    }
}

/// For `a` coprime to `n` and `x` in `coprime_residues(n)`, the product
/// `(a * x) mod n` is again in `coprime_residues(n)` (assuming `n > 0`).
/// This is the membership half of "multiplication by a coprime unit
/// permutes the units mod n".
theorem coprime_residues_mul_mem(n: Nat, a: Nat, x: Nat) {
    n != Nat.0 and a.coprime(n) and coprime_residues(n).contains(x)
        implies coprime_residues(n).contains((a * x).mod(n))
} by {
    if n != Nat.0 and a.coprime(n) and coprime_residues(n).contains(x) {
        coprime_residues_contains_imp(n, x)
        x.coprime(n)
        coprime_mul_both(n, a, x)
        (a * x).coprime(n)
        coprime_mod_imp(a * x, n)
        ((a * x).mod(n)).coprime(n)
        mod_lt(a * x, n)
        (a * x).mod(n) < n
        ((a * x).mod(n)) < n and ((a * x).mod(n)).coprime(n)
        coprime_residues_contains_intro(n, (a * x).mod(n))
        coprime_residues(n).contains((a * x).mod(n))
    }
}

/// Helper: from a congruence between two `Nat`s both below `n`, deduce equality.
theorem congr_mod_below_eq(n: Nat, x: Nat, y: Nat) {
    n != Nat.0 and x < n and y < n and x.congr_mod(y, n)
        implies x = y
} by {
    if n != Nat.0 and x < n and y < n and x.congr_mod(y, n) {
        x.mod(n) = y.mod(n)
        small_mod(x, n)
        x.mod(n) = x
        small_mod(y, n)
        y.mod(n) = y
        x = y
    }
}

/// Helper: `(a * x).mod(n) = (a * y).mod(n)` is the same as the congruence.
theorem mul_mod_eq_imp_congr(n: Nat, a: Nat, x: Nat, y: Nat) {
    (a * x).mod(n) = (a * y).mod(n)
        implies (a * x).congr_mod(a * y, n)
}

/// Multiplication by a coprime unit is injective on `[0, n)`: if
/// `(a * x) mod n = (a * y) mod n` and `x, y < n`, then `x = y`.
/// Companion of `coprime_residues_mul_mem`; together they show the
/// multiplication map is a bijection on `coprime_residues(n)`.
theorem mul_mod_inj_below(n: Nat, a: Nat, x: Nat, y: Nat) {
    n != Nat.0 and a.coprime(n) and x < n and y < n
        and (a * x).mod(n) = (a * y).mod(n)
        implies x = y
} by {
    if n != Nat.0 and a.coprime(n) and x < n and y < n
        and (a * x).mod(n) = (a * y).mod(n) {
        mul_mod_eq_imp_congr(n, a, x, y)
        (a * x).congr_mod(a * y, n)
        cancel_coprime(a, n, x, y)
        x.congr_mod(y, n)
        congr_mod_below_eq(n, x, y)
        x = y
    }
}

/// Specialisation to `coprime_residues(n)` membership.
theorem coprime_residues_mul_inj(n: Nat, a: Nat, x: Nat, y: Nat) {
    n != Nat.0 and a.coprime(n)
        and coprime_residues(n).contains(x) and coprime_residues(n).contains(y)
        and (a * x).mod(n) = (a * y).mod(n)
        implies x = y
} by {
    if n != Nat.0 and a.coprime(n)
        and coprime_residues(n).contains(x) and coprime_residues(n).contains(y)
        and (a * x).mod(n) = (a * y).mod(n) {
        coprime_residues_contains_imp(n, x)
        x < n and x.coprime(n)
        x < n
        coprime_residues_contains_imp(n, y)
        y < n and y.coprime(n)
        y < n
        mul_mod_inj_below(n, a, x, y)
        x = y
    }
}

/// Helper: consing a fresh head onto a unique list keeps it unique.
theorem cons_unique_intro(head: Nat, tail: List[Nat]) {
    not tail.contains(head) and tail.is_unique
        implies List.cons(head, tail).is_unique
} by {
    if not tail.contains(head) and tail.is_unique {
        tail.unique = tail
        // unfold cons(head, tail).unique on the not-contains branch.
        List.cons(head, tail).unique = List.cons(head, tail.unique)
        List.cons(head, tail.unique) = List.cons(head, tail)
        List.cons(head, tail).unique = List.cons(head, tail)
    }
}

/// Inductive predicate for `coprime_residues_below_unique`.
define cr_unique_pred(n: Nat) -> (Nat -> Bool) {
    function(k: Nat) {
        coprime_residues_below(n, k).is_unique
    }
}

/// Inductive step.
theorem cr_unique_step(n: Nat, k: Nat) {
    cr_unique_pred(n)(k) implies cr_unique_pred(n)(k.suc)
} by {
    if cr_unique_pred(n)(k) {
        coprime_residues_below(n, k).is_unique
        if k.coprime(n) {
            coprime_residues_below_suc_yes(n, k)
            coprime_residues_below(n, k.suc) =
                List.cons(k, coprime_residues_below(n, k))
            // k is not in coprime_residues_below(n, k) since all elements are < k.
            if coprime_residues_below(n, k).contains(k) {
                coprime_residues_below_all_below(n, k, k)
                k < k
                false
            }
            not coprime_residues_below(n, k).contains(k)
            cons_unique_intro(k, coprime_residues_below(n, k))
            List.cons(k, coprime_residues_below(n, k)).is_unique
            coprime_residues_below(n, k.suc).is_unique
        } else {
            coprime_residues_below_suc_no(n, k)
            coprime_residues_below(n, k.suc) = coprime_residues_below(n, k)
            coprime_residues_below(n, k.suc).is_unique
        }
    }
}

/// `coprime_residues_below(n, k)` has no duplicates.
theorem coprime_residues_below_unique(n: Nat, k: Nat) {
    coprime_residues_below(n, k).is_unique
} by {
    let f: Nat -> Bool = function(m: Nat) {
        coprime_residues_below(n, m).is_unique
    }
    forall(y: Nat) {
        cr_unique_pred(n)(y) = f(y)
        f(y) = cr_unique_pred(n)(y)
    }
    // Base.
    coprime_residues_below(n, Nat.0) = List.nil[Nat]
    List.nil[Nat].unique = List.nil[Nat]
    List.nil[Nat].is_unique
    f(Nat.0)
    forall(m: Nat) {
        if f(m) {
            cr_unique_pred(n)(m)
            cr_unique_step(n, m)
            cr_unique_pred(n)(m.suc)
            f(m.suc)
        }
    }
    f(k)
}

/// `coprime_residues(n)` has no duplicates.
theorem coprime_residues_unique(n: Nat) {
    coprime_residues(n).is_unique
} by {
    coprime_residues_below_unique(n, n)
    coprime_residues(n) = coprime_residues_below(n, n)
}

/// Helper: scaling `(a*b) ≡ 1` on the right by `x` gives `(a*b)*x ≡ x` (mod n).
theorem scale_inv_by_x(a: Nat, b: Nat, x: Nat, n: Nat) {
    (a * b).congr_mod(Nat.1, n) implies ((a * b) * x).congr_mod(x, n)
} by {
    if (a * b).congr_mod(Nat.1, n) {
        congr_mod_refl(x, n)
        x.congr_mod(x, n)
        congr_mod_mul(a * b, x, Nat.1, x, n)
        ((a * b) * x).congr_mod(Nat.1 * x, n)
        Nat.1 * x = x
        ((a * b) * x).congr_mod(x, n)
    }
}

/// Helper: for `b = mod_inv(a, n)` and any `x`, `(a * (b * x)).congr_mod(x, n)`.
/// Sets up the witness `(b * x).mod(n)` for the surjectivity of multiplication
/// by a unit.
theorem mul_inv_x_back(a: Nat, x: Nat, n: Nat) {
    a.coprime(n)
        implies (a * (mod_inv(a, n) * x)).congr_mod(x, n)
} by {
    if a.coprime(n) {
        mod_inv_mul_congr_one(a, n)
        (a * mod_inv(a, n)).congr_mod(Nat.1, n)
        scale_inv_by_x(a, mod_inv(a, n), x, n)
        ((a * mod_inv(a, n)) * x).congr_mod(x, n)
        a * (mod_inv(a, n) * x) = (a * mod_inv(a, n)) * x
        (a * (mod_inv(a, n) * x)).congr_mod(x, n)
    }
}

/// Helper: scaling a value by `(b * x).mod(n)` versus `b * x` is a congruence.
theorem mul_mod_inner_congr(a: Nat, b: Nat, x: Nat, n: Nat) {
    (a * (b * x).mod(n)).congr_mod(a * (b * x), n)
} by {
    mod_congr_mod_self(b * x, n)
    (b * x).mod(n).congr_mod(b * x, n)
    congr_mod_refl(a, n)
    a.congr_mod(a, n)
    congr_mod_mul(a, (b * x).mod(n), a, b * x, n)
}

/// Surjectivity half: for `n > 0`, `a` coprime to `n`, and `x` in
/// `coprime_residues(n)`, there is a `y` in `coprime_residues(n)` with
/// `(a * y).mod(n) = x`. Witness: `y = (mod_inv(a, n) * x).mod(n)`.
theorem coprime_residues_mul_surj(n: Nat, a: Nat, x: Nat) {
    n != Nat.0 and a.coprime(n) and coprime_residues(n).contains(x)
        implies exists(y: Nat) {
            coprime_residues(n).contains(y) and (a * y).mod(n) = x
        }
} by {
    if n != Nat.0 and a.coprime(n) and coprime_residues(n).contains(x) {
        coprime_residues_contains_imp(n, x)
        x < n
        x.coprime(n)
        let b: Nat = mod_inv(a, n)
        mod_inv_coprime(a, n)
        b.coprime(n)
        let y: Nat = (b * x).mod(n)
        // y < n since n > 0.
        mod_lt(b * x, n)
        y < n
        // y is coprime to n.
        coprime_mul_both(n, b, x)
        (b * x).coprime(n)
        coprime_mod_imp(b * x, n)
        ((b * x).mod(n)).coprime(n)
        y.coprime(n)
        // y is in coprime_residues(n).
        coprime_residues_contains_intro(n, y)
        coprime_residues(n).contains(y)
        // (a * y).mod(n) = x.
        mul_mod_inner_congr(a, b, x, n)
        (a * y).congr_mod(a * (b * x), n)
        mul_inv_x_back(a, x, n)
        (a * (b * x)).congr_mod(x, n)
        congr_mod_trans(a * y, a * (b * x), x, n)
        (a * y).congr_mod(x, n)
        // congr_mod is mod equality.
        (a * y).mod(n) = x.mod(n)
        small_mod(x, n)
        x.mod(n) = x
        (a * y).mod(n) = x
        coprime_residues(n).contains(y) and (a * y).mod(n) = x
        exists(y0: Nat) {
            coprime_residues(n).contains(y0) and (a * y0).mod(n) = x
        }
    }
}

/// Top-level point function `x -> (a * x).mod(n)` so that `map` clients
/// don't have to inline the lambda. Using a named function rather than an
/// anonymous one inside `define mul_mod_residues` keeps the equational
/// `mul_mod_residues(n, a) = map(coprime_residues(n), mul_mod_fn(n, a))`
/// reflexive under `acorn check`'s certificate replay.
define mul_mod_fn(n: Nat, a: Nat) -> (Nat -> Nat) {
    function(x: Nat) { (a * x).mod(n) }
}

/// The map `x -> (a * x).mod(n)` applied to every element of
/// `coprime_residues(n)`. By multiplication-by-unit membership and
/// surjectivity, this list has the same elements as `coprime_residues(n)`.
define mul_mod_residues(n: Nat, a: Nat) -> List[Nat] {
    map(coprime_residues(n), mul_mod_fn(n, a))
}

/// Length of `mul_mod_residues` matches `coprime_residues`, which is
/// `nat_totient(n)`.
theorem mul_mod_residues_length(n: Nat, a: Nat) {
    mul_mod_residues(n, a).length = n.totient
} by {
    map_length(coprime_residues(n), mul_mod_fn(n, a))
    map(coprime_residues(n), mul_mod_fn(n, a)).length = coprime_residues(n).length
    coprime_residues_length(n)
    coprime_residues(n).length = n.totient
}

/// Forward inclusion: every element of `mul_mod_residues(n, a)` lies in
/// `coprime_residues(n)` (assuming `n > 0` and `a` coprime to `n`).
theorem mul_mod_residues_in_coprime(n: Nat, a: Nat, y: Nat) {
    n != Nat.0 and a.coprime(n) and mul_mod_residues(n, a).contains(y)
        implies coprime_residues(n).contains(y)
} by {
    if n != Nat.0 and a.coprime(n) and mul_mod_residues(n, a).contains(y) {
        map(coprime_residues(n), mul_mod_fn(n, a)).contains(y)
        map_contains(coprime_residues(n), mul_mod_fn(n, a), y)
        let x: Nat satisfy {
            coprime_residues(n).contains(x) and mul_mod_fn(n, a)(x) = y
        }
        coprime_residues(n).contains(x)
        mul_mod_fn(n, a)(x) = (a * x).mod(n)
        (a * x).mod(n) = y
        coprime_residues_mul_mem(n, a, x)
        coprime_residues(n).contains((a * x).mod(n))
        coprime_residues(n).contains(y)
    }
}

/// Backward inclusion: every element of `coprime_residues(n)` is in
/// `mul_mod_residues(n, a)` (assuming `n > 0` and `a` coprime to `n`).
theorem coprime_in_mul_mod_residues(n: Nat, a: Nat, y: Nat) {
    n != Nat.0 and a.coprime(n) and coprime_residues(n).contains(y)
        implies mul_mod_residues(n, a).contains(y)
} by {
    if n != Nat.0 and a.coprime(n) and coprime_residues(n).contains(y) {
        coprime_residues_mul_surj(n, a, y)
        let x: Nat satisfy {
            coprime_residues(n).contains(x) and (a * x).mod(n) = y
        }
        coprime_residues(n).contains(x)
        mul_mod_fn(n, a)(x) = (a * x).mod(n)
        mul_mod_fn(n, a)(x) = y
        map_contains_of_contains(coprime_residues(n), mul_mod_fn(n, a), x)
        map(coprime_residues(n), mul_mod_fn(n, a)).contains(mul_mod_fn(n, a)(x))
        map(coprime_residues(n), mul_mod_fn(n, a)).contains(y)
    }
}


/// Inductive predicate for `mul_mod_residues_below_unique`: for `n != 0` and
/// `a` coprime to `n`, the image of `coprime_residues_below(n, k)` under
/// `mul_mod_fn(n, a)` is `is_unique`, provided `k <= n`.
define mul_mod_residues_below_unique_pred(n: Nat, a: Nat) -> (Nat -> Bool) {
    function(k: Nat) {
        k <= n implies
            map(coprime_residues_below(n, k), mul_mod_fn(n, a)).is_unique
    }
}

/// Inductive step: walk past one more candidate, preserving uniqueness of the
/// mapped list.
theorem mul_mod_residues_below_unique_step(n: Nat, a: Nat, k: Nat) {
    n != Nat.0 and a.coprime(n)
        and mul_mod_residues_below_unique_pred(n, a)(k)
        implies mul_mod_residues_below_unique_pred(n, a)(k.suc)
} by {
    if n != Nat.0 and a.coprime(n)
        and mul_mod_residues_below_unique_pred(n, a)(k) {
        if k.suc <= n {
            k <= n
            k < n
            // IH: map(below(n, k), f) is unique.
            map(coprime_residues_below(n, k), mul_mod_fn(n, a)).is_unique
            if k.coprime(n) {
                coprime_residues_below_suc_yes(n, k)
                coprime_residues_below(n, k.suc) =
                    List.cons(k, coprime_residues_below(n, k))
                map(List.cons(k, coprime_residues_below(n, k)), mul_mod_fn(n, a)) =
                    List.cons(mul_mod_fn(n, a)(k),
                              map(coprime_residues_below(n, k), mul_mod_fn(n, a)))
                map(coprime_residues_below(n, k.suc), mul_mod_fn(n, a)) =
                    List.cons(mul_mod_fn(n, a)(k),
                              map(coprime_residues_below(n, k), mul_mod_fn(n, a)))
                // Goal: cons(f(k), tail) is unique.
                // Need: f(k) not in tail.
                if map(coprime_residues_below(n, k), mul_mod_fn(n, a)).contains(
                    mul_mod_fn(n, a)(k)) {
                    map_contains(coprime_residues_below(n, k), mul_mod_fn(n, a),
                                 mul_mod_fn(n, a)(k))
                    let x: Nat satisfy {
                        coprime_residues_below(n, k).contains(x) and
                            mul_mod_fn(n, a)(x) = mul_mod_fn(n, a)(k)
                    }
                    coprime_residues_below(n, k).contains(x)
                    coprime_residues_below_all_below(n, k, x)
                    x < k
                    // x and k both in coprime_residues(n).
                    // First, x: x < k < n, x.coprime(n).
                    coprime_residues_below_all_coprime(n, k, x)
                    x.coprime(n)
                    x < n
                    coprime_residues_contains_intro(n, x)
                    coprime_residues(n).contains(x)
                    // k: k < n, k.coprime(n).
                    coprime_residues_contains_intro(n, k)
                    coprime_residues(n).contains(k)
                    // mul_mod_fn equals mul_mod.
                    mul_mod_fn(n, a)(x) = (a * x).mod(n)
                    mul_mod_fn(n, a)(k) = (a * k).mod(n)
                    (a * x).mod(n) = (a * k).mod(n)
                    coprime_residues_mul_inj(n, a, x, k)
                    x = k
                    // x < k contradicts x = k.
                    x < x
                    false
                }
                not map(coprime_residues_below(n, k), mul_mod_fn(n, a)).contains(
                    mul_mod_fn(n, a)(k))
                let mapped: List[Nat] = map(coprime_residues_below(n, k), mul_mod_fn(n, a))
                not mapped.contains(mul_mod_fn(n, a)(k))
                mapped.is_unique
                cons_unique_intro(mul_mod_fn(n, a)(k), mapped)
                List.cons(mul_mod_fn(n, a)(k), mapped).is_unique
                map(coprime_residues_below(n, k.suc), mul_mod_fn(n, a)).is_unique
            } else {
                coprime_residues_below_suc_no(n, k)
                coprime_residues_below(n, k.suc) = coprime_residues_below(n, k)
                map(coprime_residues_below(n, k.suc), mul_mod_fn(n, a)) =
                    map(coprime_residues_below(n, k), mul_mod_fn(n, a))
                map(coprime_residues_below(n, k.suc), mul_mod_fn(n, a)).is_unique
            }
        }
    }
}

/// Inner induction over `k`.
theorem mul_mod_residues_below_unique_run(n: Nat, a: Nat, k: Nat) {
    n != Nat.0 and a.coprime(n) implies mul_mod_residues_below_unique_pred(n, a)(k)
} by {
    if n != Nat.0 and a.coprime(n) {
        let f: Nat -> Bool = function(m: Nat) {
            m <= n implies
                map(coprime_residues_below(n, m), mul_mod_fn(n, a)).is_unique
        }
        forall(y: Nat) {
            mul_mod_residues_below_unique_pred(n, a)(y) = f(y)
            f(y) = mul_mod_residues_below_unique_pred(n, a)(y)
        }
        // Base: k = 0.
        coprime_residues_below(n, Nat.0) = List.nil[Nat]
        map(List.nil[Nat], mul_mod_fn(n, a)) = List.nil[Nat]
        List.nil[Nat].unique = List.nil[Nat]
        List.nil[Nat].is_unique
        f(Nat.0)
        forall(m: Nat) {
            if f(m) {
                mul_mod_residues_below_unique_pred(n, a)(m)
                mul_mod_residues_below_unique_step(n, a, m)
                mul_mod_residues_below_unique_pred(n, a)(m.suc)
                f(m.suc)
            }
        }
        f(k)
        mul_mod_residues_below_unique_pred(n, a)(k)
    }
}

/// `map(coprime_residues_below(n, k), mul_mod_fn(n, a))` is unique for
/// `k <= n` (when `n != 0` and `a` coprime to `n`).
theorem mul_mod_residues_below_unique(n: Nat, a: Nat, k: Nat) {
    n != Nat.0 and a.coprime(n) and k <= n
        implies map(coprime_residues_below(n, k), mul_mod_fn(n, a)).is_unique
} by {
    if n != Nat.0 and a.coprime(n) and k <= n {
        mul_mod_residues_below_unique_run(n, a, k)
        mul_mod_residues_below_unique_pred(n, a)(k)
    }
}

/// `mul_mod_residues(n, a)` has no duplicates whenever `n > 0` and `a` is
/// coprime to `n`.
theorem mul_mod_residues_unique(n: Nat, a: Nat) {
    n != Nat.0 and a.coprime(n) implies mul_mod_residues(n, a).is_unique
} by {
    if n != Nat.0 and a.coprime(n) {
        n <= n
        mul_mod_residues_below_unique(n, a, n)
        let mapped: List[Nat] = map(coprime_residues_below(n, n), mul_mod_fn(n, a))
        mapped.is_unique
        coprime_residues(n) = coprime_residues_below(n, n)
        map(coprime_residues(n), mul_mod_fn(n, a)) = mapped
        mul_mod_residues(n, a) = map(coprime_residues(n), mul_mod_fn(n, a))
        mul_mod_residues(n, a) = mapped
        mul_mod_residues(n, a).is_unique
    }
}

/// Helper: in a unique list, count is exactly one when contained.
theorem unique_count_one(list: List[Nat], item: Nat) {
    list.is_unique and list.contains(item) implies list.count(item) = Nat.1
} by {
    if list.is_unique and list.contains(item) {
        list_contains_implies_count_geq_one(list, item)
        list.count(item) >= Nat.1
        unique_implies_no_duplicate(list, item)
        list.count(item) <= Nat.1
        list.count(item) = Nat.1
    }
}

/// Helper: in any list, count is zero when not contained.
theorem not_contains_count_zero(list: List[Nat], item: Nat) {
    not list.contains(item) implies list.count(item) = Nat.0
} by {
    if not list.contains(item) {
        list_not_contains_impl_count_zero(list, item)
    }
}

/// `is_permutation(mul_mod_residues(n, a), coprime_residues(n))` for `n > 0`
/// and `a` coprime to `n`. Combines the bidirectional membership with
/// uniqueness on both sides: each element contributes count `0` or `1`, and
/// the contains-statuses agree.
theorem mul_mod_residues_is_permutation(n: Nat, a: Nat) {
    n != Nat.0 and a.coprime(n)
        implies is_permutation(mul_mod_residues(n, a), coprime_residues(n))
} by {
    if n != Nat.0 and a.coprime(n) {
        mul_mod_residues_unique(n, a)
        mul_mod_residues(n, a).is_unique
        coprime_residues_unique(n)
        coprime_residues(n).is_unique
        forall(y: Nat) {
            if mul_mod_residues(n, a).contains(y) {
                mul_mod_residues_in_coprime(n, a, y)
                coprime_residues(n).contains(y)
                unique_count_one(mul_mod_residues(n, a), y)
                unique_count_one(coprime_residues(n), y)
                mul_mod_residues(n, a).count(y) = Nat.1
                coprime_residues(n).count(y) = Nat.1
                mul_mod_residues(n, a).count(y) = coprime_residues(n).count(y)
            } else {
                not mul_mod_residues(n, a).contains(y)
                if coprime_residues(n).contains(y) {
                    coprime_in_mul_mod_residues(n, a, y)
                    mul_mod_residues(n, a).contains(y)
                    false
                }
                not coprime_residues(n).contains(y)
                not_contains_count_zero(mul_mod_residues(n, a), y)
                not_contains_count_zero(coprime_residues(n), y)
                mul_mod_residues(n, a).count(y) = Nat.0
                coprime_residues(n).count(y) = Nat.0
                mul_mod_residues(n, a).count(y) = coprime_residues(n).count(y)
            }
        }
    }
}

/// Top-level scalar-multiplication function for use with `map`. Like
/// `mul_mod_fn`, naming this avoids inline-lambda issues in `acorn check`'s
/// certificate replay.
define scalar_mul_fn(a: Nat) -> (Nat -> Nat) {
    function(x: Nat) { a * x }
}

/// Inductive predicate for `product_map_scalar`.
define product_map_scalar_pred(a: Nat) -> (List[Nat] -> Bool) {
    function(l: List[Nat]) {
        product[Nat](map(l, scalar_mul_fn(a))) = a.pow(l.length) * product[Nat](l)
    }
}

/// Base case: empty list. Both sides reduce to `Nat.1`.
theorem product_map_scalar_nil(a: Nat) {
    product_map_scalar_pred(a)(List.nil[Nat])
} by {
    map(List.nil[Nat], scalar_mul_fn(a)) = List.nil[Nat]
    product[Nat](List.nil[Nat]) = Nat.1
    List.nil[Nat].length = Nat.0
    exp_zero(a)
    a.pow(Nat.0) = Nat.1
    a.pow(List.nil[Nat].length) * product[Nat](List.nil[Nat]) = Nat.1 * Nat.1
    Nat.1 * Nat.1 = Nat.1
}

/// Inductive step.
theorem product_map_scalar_cons(a: Nat, head: Nat, tail: List[Nat]) {
    product_map_scalar_pred(a)(tail)
        implies product_map_scalar_pred(a)(List.cons(head, tail))
} by {
    if product_map_scalar_pred(a)(tail) {
        product[Nat](map(tail, scalar_mul_fn(a))) = a.pow(tail.length) * product[Nat](tail)
        // map(cons(h, t), f) = cons(f(h), map(t, f)) = cons(a*h, map(t, f)).
        scalar_mul_fn(a)(head) = a * head
        map(List.cons(head, tail), scalar_mul_fn(a)) =
            List.cons(scalar_mul_fn(a)(head), map(tail, scalar_mul_fn(a)))
        map(List.cons(head, tail), scalar_mul_fn(a)) =
            List.cons(a * head, map(tail, scalar_mul_fn(a)))
        // product over cons.
        product[Nat](List.cons(a * head, map(tail, scalar_mul_fn(a)))) =
            (a * head) * product[Nat](map(tail, scalar_mul_fn(a)))
        // Substitute IH.
        (a * head) * product[Nat](map(tail, scalar_mul_fn(a))) =
            (a * head) * (a.pow(tail.length) * product[Nat](tail))
        // Rearrange to a^(tail.length.suc) * (head * product(tail)).
        (a * head) * (a.pow(tail.length) * product[Nat](tail)) =
            a * head * a.pow(tail.length) * product[Nat](tail)
        a * head * a.pow(tail.length) = a * a.pow(tail.length) * head
        a * a.pow(tail.length) = a.pow(tail.length) * a
        a.pow(tail.length) * a = a.pow(tail.length + Nat.1)
        tail.length + Nat.1 = tail.length.suc
        a.pow(tail.length) * a = a.pow(tail.length.suc)
        a * a.pow(tail.length) = a.pow(tail.length.suc)
        a * head * a.pow(tail.length) = a.pow(tail.length.suc) * head
        // Combine.
        a * head * a.pow(tail.length) * product[Nat](tail) =
            a.pow(tail.length.suc) * head * product[Nat](tail)
        a.pow(tail.length.suc) * head * product[Nat](tail) =
            a.pow(tail.length.suc) * (head * product[Nat](tail))
        // RHS shape.
        List.cons(head, tail).length = tail.length.suc
        product[Nat](List.cons(head, tail)) = head * product[Nat](tail)
        a.pow(List.cons(head, tail).length) * product[Nat](List.cons(head, tail)) =
            a.pow(tail.length.suc) * (head * product[Nat](tail))
        // Wrap up.
        product[Nat](map(List.cons(head, tail), scalar_mul_fn(a))) =
            a.pow(List.cons(head, tail).length) * product[Nat](List.cons(head, tail))
    }
}

/// `product(map(L, λx. a*x)) = a^|L| * product(L)` for Nat lists.
/// Pulls the constant factor `a` out of every term in the list product,
/// raised to the list length. Building block for Euler-style arguments.
theorem product_map_scalar(a: Nat, l: List[Nat]) {
    product[Nat](map(l, scalar_mul_fn(a))) = a.pow(l.length) * product[Nat](l)
} by {
    define p(x: List[Nat]) -> Bool {
        product[Nat](map(x, scalar_mul_fn(a))) = a.pow(x.length) * product[Nat](x)
    }
    forall(y: List[Nat]) {
        product_map_scalar_pred(a)(y) = p(y)
        p(y) = product_map_scalar_pred(a)(y)
    }
    product_map_scalar_nil(a)
    p(List.nil[Nat])
    forall(head: Nat, tail: List[Nat]) {
        if p(tail) {
            product_map_scalar_pred(a)(tail)
            product_map_scalar_cons(a, head, tail)
            product_map_scalar_pred(a)(List.cons(head, tail))
            p(List.cons(head, tail))
        }
    }
    List.induction(function(z: List[Nat]) { p(z) })
    forall(z: List[Nat]) { p(z) }
    p(l)
}

/// Inductive predicate for `product_coprime_of_all`.
define product_coprime_pred(n: Nat) -> (List[Nat] -> Bool) {
    function(l: List[Nat]) {
        (forall(x: Nat) { l.contains(x) implies x.coprime(n) })
            implies product[Nat](l).coprime(n)
    }
}

/// Bridge: unfolds the predicate to its underlying implication.
theorem product_coprime_pred_apply(n: Nat, l: List[Nat]) {
    product_coprime_pred(n)(l)
        and (forall(x: Nat) { l.contains(x) implies x.coprime(n) })
        implies product[Nat](l).coprime(n)
}

/// Base case: empty list. `product(nil) = 1` is coprime to `n`.
theorem product_coprime_nil(n: Nat) {
    product_coprime_pred(n)(List.nil[Nat])
} by {
    product[Nat](List.nil[Nat]) = Nat.1
    coprime_one_left(n)
    Nat.1.coprime(n)
}

/// Inductive step.
theorem product_coprime_cons(n: Nat, head: Nat, tail: List[Nat]) {
    product_coprime_pred(n)(tail)
        implies product_coprime_pred(n)(List.cons(head, tail))
} by {
    if product_coprime_pred(n)(tail) {
        if (forall(x: Nat) {
            List.cons(head, tail).contains(x) implies x.coprime(n)
        }) {
            // head is coprime to n.
            List.cons(head, tail).contains(head)
            head.coprime(n)
            // every element of tail is coprime to n.
            forall(x: Nat) {
                if tail.contains(x) {
                    List.cons(head, tail).contains(x)
                    x.coprime(n)
                }
            }
            // Apply IH to get product(tail) coprime n.
            product_coprime_pred_apply(n, tail)
            product[Nat](tail).coprime(n)
            // product(cons(head, tail)) = head * product(tail).
            product[Nat](List.cons(head, tail)) = head * product[Nat](tail)
            coprime_mul_both(n, head, product[Nat](tail))
            (head * product[Nat](tail)).coprime(n)
            product[Nat](List.cons(head, tail)).coprime(n)
        }
    }
}

/// `product[Nat](L).coprime(n)` whenever every element of `L` is coprime to
/// `n`.
theorem product_coprime_of_all(n: Nat, l: List[Nat]) {
    (forall(x: Nat) { l.contains(x) implies x.coprime(n) })
        implies product[Nat](l).coprime(n)
} by {
    define p(x: List[Nat]) -> Bool { product_coprime_pred(n)(x) }
    product_coprime_nil(n)
    p(List.nil[Nat])
    forall(head: Nat, tail: List[Nat]) {
        if p(tail) {
            product_coprime_cons(n, head, tail)
            p(List.cons(head, tail))
        }
    }
    List.induction(function(z: List[Nat]) { p(z) })
    forall(z: List[Nat]) { p(z) }
    p(l)
    product_coprime_pred(n)(l)
    if (forall(x: Nat) { l.contains(x) implies x.coprime(n) }) {
        product_coprime_pred_apply(n, l)
    }
}

/// `product(coprime_residues(n)).coprime(n)`. Specialisation of
/// `product_coprime_of_all` to the units list — every element is coprime to
/// `n` by construction (`coprime_residues_all_coprime`), so their product is
/// too.
theorem product_coprime_residues_coprime(n: Nat) {
    product[Nat](coprime_residues(n)).coprime(n)
} by {
    forall(x: Nat) {
        if coprime_residues(n).contains(x) {
            coprime_residues_all_coprime(n, x)
            x.coprime(n)
        }
    }
    product_coprime_of_all(n, coprime_residues(n))
}


/// Helper: rewrite `product(map(cons(h, t), f))` as `f(h) * product(map(t, f))`.
theorem product_map_cons(f: Nat -> Nat, head: Nat, tail: List[Nat]) {
    product[Nat](map(List.cons(head, tail), f)) =
        f(head) * product[Nat](map(tail, f))
} by {
    map(List.cons(head, tail), f) = List.cons(f(head), map(tail, f))
    product[Nat](List.cons(f(head), map(tail, f))) =
        f(head) * product[Nat](map(tail, f))
}

/// Inductive predicate for `product_mul_mod_congr_scalar`: at each list `l`,
/// the product of `map(l, mul_mod_fn(n, a))` is congruent modulo `n` to the
/// product of `map(l, scalar_mul_fn(a))`.
define pmmcs_pred(n: Nat, a: Nat) -> (List[Nat] -> Bool) {
    function(l: List[Nat]) {
        product[Nat](map(l, mul_mod_fn(n, a))).congr_mod(
            product[Nat](map(l, scalar_mul_fn(a))), n)
    }
}

/// Base case: empty list. Both products are `1`.
theorem pmmcs_nil(n: Nat, a: Nat) {
    pmmcs_pred(n, a)(List.nil[Nat])
} by {
    map(List.nil[Nat], mul_mod_fn(n, a)) = List.nil[Nat]
    map(List.nil[Nat], scalar_mul_fn(a)) = List.nil[Nat]
    product[Nat](List.nil[Nat]) = Nat.1
    congr_mod_refl(Nat.1, n)
}

/// Inductive step.
theorem pmmcs_cons(n: Nat, a: Nat, head: Nat, tail: List[Nat]) {
    pmmcs_pred(n, a)(tail) implies pmmcs_pred(n, a)(List.cons(head, tail))
} by {
    if pmmcs_pred(n, a)(tail) {
        product[Nat](map(tail, mul_mod_fn(n, a))).congr_mod(
            product[Nat](map(tail, scalar_mul_fn(a))), n)
        product_map_cons(mul_mod_fn(n, a), head, tail)
        product[Nat](map(List.cons(head, tail), mul_mod_fn(n, a))) =
            mul_mod_fn(n, a)(head) * product[Nat](map(tail, mul_mod_fn(n, a)))
        product_map_cons(scalar_mul_fn(a), head, tail)
        product[Nat](map(List.cons(head, tail), scalar_mul_fn(a))) =
            scalar_mul_fn(a)(head) * product[Nat](map(tail, scalar_mul_fn(a)))
        // Per-element congruence: (a * head).mod(n) ≡ a * head  (mod n).
        mul_mod_fn(n, a)(head) = (a * head).mod(n)
        scalar_mul_fn(a)(head) = a * head
        mod_congr_mod_self(a * head, n)
        (a * head).mod(n).congr_mod(a * head, n)
        mul_mod_fn(n, a)(head).congr_mod(scalar_mul_fn(a)(head), n)
        // Multiply the head congruence with the tail congruence.
        congr_mod_mul(mul_mod_fn(n, a)(head),
                      product[Nat](map(tail, mul_mod_fn(n, a))),
                      scalar_mul_fn(a)(head),
                      product[Nat](map(tail, scalar_mul_fn(a))), n)
        (mul_mod_fn(n, a)(head) * product[Nat](map(tail, mul_mod_fn(n, a)))).congr_mod(
            scalar_mul_fn(a)(head) * product[Nat](map(tail, scalar_mul_fn(a))), n)
        product[Nat](map(List.cons(head, tail), mul_mod_fn(n, a))).congr_mod(
            product[Nat](map(List.cons(head, tail), scalar_mul_fn(a))), n)
    }
}

/// For any list of naturals, the product of its image under `mul_mod_fn(n, a)`
/// is congruent mod `n` to the product of its image under `scalar_mul_fn(a)`.
/// Bridges the modded multiplication used in `mul_mod_residues` with plain
/// multiplication, for use in the Euler product argument.
theorem product_mul_mod_congr_scalar(n: Nat, a: Nat, l: List[Nat]) {
    product[Nat](map(l, mul_mod_fn(n, a))).congr_mod(
        product[Nat](map(l, scalar_mul_fn(a))), n)
} by {
    define p(x: List[Nat]) -> Bool { pmmcs_pred(n, a)(x) }
    pmmcs_nil(n, a)
    p(List.nil[Nat])
    forall(head: Nat, tail: List[Nat]) {
        if p(tail) {
            pmmcs_cons(n, a, head, tail)
            p(List.cons(head, tail))
        }
    }
    List.induction(function(z: List[Nat]) { p(z) })
    forall(z: List[Nat]) { p(z) }
    p(l)
    pmmcs_pred(n, a)(l)
}

/// Euler's theorem: for `n > 0` and `a` coprime to `n`,
///   `a.pow(n.totient) ≡ 1  (mod n)`.
/// Generalises `fermat_euler` from prime moduli and `euler_pq` from `p * q`
/// to arbitrary positive moduli, using the coprime-residues permutation
/// argument.
theorem fermat_euler_general(n: Nat, a: Nat) {
    n != Nat.0 and a.coprime(n)
        implies a.pow(n.totient).congr_mod(Nat.1, n)
} by {
    if n != Nat.0 and a.coprime(n) {
        let r: List[Nat] = coprime_residues(n)
        // Permutation step: product over `mul_mod_residues(n, a)` equals
        // product over the units list.
        mul_mod_residues_is_permutation(n, a)
        is_permutation(mul_mod_residues(n, a), r)
        permutation_preserves_product[Nat](mul_mod_residues(n, a), r)
        product[Nat](mul_mod_residues(n, a)) = product[Nat](r)
        mul_mod_residues(n, a) = map(r, mul_mod_fn(n, a))
        product[Nat](map(r, mul_mod_fn(n, a))) = product[Nat](r)
        // Element-wise congruence step: bridge to `scalar_mul_fn`.
        product_mul_mod_congr_scalar(n, a, r)
        product[Nat](map(r, mul_mod_fn(n, a))).congr_mod(
            product[Nat](map(r, scalar_mul_fn(a))), n)
        product[Nat](r).congr_mod(product[Nat](map(r, scalar_mul_fn(a))), n)
        // Pull the scalar out.
        product_map_scalar(a, r)
        product[Nat](map(r, scalar_mul_fn(a))) = a.pow(r.length) * product[Nat](r)
        coprime_residues_length(n)
        r.length = n.totient
        a.pow(r.length) = a.pow(n.totient)
        a.pow(r.length) * product[Nat](r) = a.pow(n.totient) * product[Nat](r)
        product[Nat](map(r, scalar_mul_fn(a))) = a.pow(n.totient) * product[Nat](r)
        product[Nat](r).congr_mod(a.pow(n.totient) * product[Nat](r), n)
        // Reorient and reshape into `P * a^φ ≡ P * 1`.
        congr_mod_symm(
            product[Nat](r), a.pow(n.totient) * product[Nat](r), n)
        (a.pow(n.totient) * product[Nat](r)).congr_mod(product[Nat](r), n)
        a.pow(n.totient) * product[Nat](r) =
            product[Nat](r) * a.pow(n.totient)
        product[Nat](r) * Nat.1 = product[Nat](r)
        (product[Nat](r) * a.pow(n.totient)).congr_mod(
            product[Nat](r) * Nat.1, n)
        // Cancel the unit product.
        product_coprime_residues_coprime(n)
        product[Nat](r).coprime(n)
        cancel_coprime(product[Nat](r), n, a.pow(n.totient), Nat.1)
        a.pow(n.totient).congr_mod(Nat.1, n)
    }
}

/// Inductive predicate for `product_map_mul_mod`: replacing each list entry
/// `x` by `(a * x).mod(n)` only changes the list product by a power of `a`,
/// modulo `n`.
define product_map_mul_mod_pred(n: Nat, a: Nat) -> (List[Nat] -> Bool) {
    function(l: List[Nat]) {
        product[Nat](map(l, mul_mod_fn(n, a))).congr_mod(
            a.pow(l.length) * product[Nat](l), n)
    }
}

/// Base case of `product_map_mul_mod`: the empty list. Both sides reduce to
/// `Nat.1`.
theorem product_map_mul_mod_nil(n: Nat, a: Nat) {
    product_map_mul_mod_pred(n, a)(List.nil[Nat])
} by {
    map(List.nil[Nat], mul_mod_fn(n, a)) = List.nil[Nat]
    product[Nat](List.nil[Nat]) = Nat.1
    List.nil[Nat].length = Nat.0
    exp_zero(a)
    a.pow(Nat.0) = Nat.1
    a.pow(List.nil[Nat].length) * product[Nat](List.nil[Nat]) = Nat.1
    congr_mod_refl(Nat.1, n)
    product[Nat](map(List.nil[Nat], mul_mod_fn(n, a))).congr_mod(
        a.pow(List.nil[Nat].length) * product[Nat](List.nil[Nat]), n)
}

/// Inductive step of `product_map_mul_mod`.
theorem product_map_mul_mod_cons(n: Nat, a: Nat, head: Nat, tail: List[Nat]) {
    product_map_mul_mod_pred(n, a)(tail)
        implies product_map_mul_mod_pred(n, a)(List.cons(head, tail))
} by {
    if product_map_mul_mod_pred(n, a)(tail) {
        // Inductive hypothesis.
        product[Nat](map(tail, mul_mod_fn(n, a))).congr_mod(
            a.pow(tail.length) * product[Nat](tail), n)
        // Unfold the cons-side product.
        product_map_cons(mul_mod_fn(n, a), head, tail)
        product[Nat](map(List.cons(head, tail), mul_mod_fn(n, a))) =
            mul_mod_fn(n, a)(head) * product[Nat](map(tail, mul_mod_fn(n, a)))
        mul_mod_fn(n, a)(head) = (a * head).mod(n)
        // `(a*head).mod(n)` is congruent to `a*head` modulo `n`.
        mod_congr_mod_self(a * head, n)
        (a * head).mod(n).congr_mod(a * head, n)
        // Combine with the inductive hypothesis multiplicatively.
        congr_mod_mul((a * head).mod(n), a * head,
                      product[Nat](map(tail, mul_mod_fn(n, a))),
                      a.pow(tail.length) * product[Nat](tail), n)
        ((a * head).mod(n) * product[Nat](map(tail, mul_mod_fn(n, a)))).congr_mod(
            (a * head) * (a.pow(tail.length) * product[Nat](tail)), n)
        // The cons-side product equals the LHS of the congruence.
        product[Nat](map(List.cons(head, tail), mul_mod_fn(n, a))).congr_mod(
            (a * head) * (a.pow(tail.length) * product[Nat](tail)), n)
        // Rewrite the RHS in terms of `a.pow(|cons|) * product(cons)`.
        List.cons(head, tail).length = tail.length.suc
        tail.length.suc = tail.length + Nat.1
        exp_add(a, tail.length, Nat.1)
        a.pow(tail.length + Nat.1) = a.pow(tail.length) * a.pow(Nat.1)
        exp_one(a)
        a.pow(Nat.1) = a
        a.pow(List.cons(head, tail).length) = a.pow(tail.length) * a
        product[Nat](List.cons(head, tail)) = head * product[Nat](tail)
        a.pow(List.cons(head, tail).length) * product[Nat](List.cons(head, tail)) = (a.pow(tail.length) * a) * (head * product[Nat](tail))
        // Rearrange (a^|t| * a) * (h * P) into (a*h) * (a^|t| * P) step by step.
        (a.pow(tail.length) * a) * (head * product[Nat](tail)) = a.pow(tail.length) * (a * (head * product[Nat](tail)))
        a * (head * product[Nat](tail)) = (a * head) * product[Nat](tail)
        a.pow(tail.length) * (a * (head * product[Nat](tail))) = a.pow(tail.length) * ((a * head) * product[Nat](tail))
        a.pow(tail.length) * ((a * head) * product[Nat](tail)) = (a * head) * (a.pow(tail.length) * product[Nat](tail))
        (a.pow(tail.length) * a) * (head * product[Nat](tail)) = (a * head) * (a.pow(tail.length) * product[Nat](tail))
        a.pow(List.cons(head, tail).length) * product[Nat](List.cons(head, tail)) = (a * head) * (a.pow(tail.length) * product[Nat](tail))
        product[Nat](map(List.cons(head, tail), mul_mod_fn(n, a))).congr_mod(
            a.pow(List.cons(head, tail).length) * product[Nat](List.cons(head, tail)), n)
    }
}

/// `product(map(L, x -> (a*x).mod(n))) ≡ a^|L| * product(L) (mod n)` for any
/// list of naturals `L`. Together with the permutation argument this drives
/// the general Euler's theorem.
theorem product_map_mul_mod(n: Nat, a: Nat, l: List[Nat]) {
    product[Nat](map(l, mul_mod_fn(n, a))).congr_mod(
        a.pow(l.length) * product[Nat](l), n)
} by {
    define p(x: List[Nat]) -> Bool {
        product[Nat](map(x, mul_mod_fn(n, a))).congr_mod(
            a.pow(x.length) * product[Nat](x), n)
    }
    forall(y: List[Nat]) {
        product_map_mul_mod_pred(n, a)(y) = p(y)
        p(y) = product_map_mul_mod_pred(n, a)(y)
    }
    product_map_mul_mod_nil(n, a)
    p(List.nil[Nat])
    forall(head: Nat, tail: List[Nat]) {
        if p(tail) {
            product_map_mul_mod_pred(n, a)(tail)
            product_map_mul_mod_cons(n, a, head, tail)
            product_map_mul_mod_pred(n, a)(List.cons(head, tail))
            p(List.cons(head, tail))
        }
    }
    List.induction(function(z: List[Nat]) { p(z) })
    forall(z: List[Nat]) { p(z) }
    p(l)
}

/// Specialisation to `coprime_residues(n)`: the product of
/// `mul_mod_residues(n, a)` is congruent to `a^φ(n) * product(coprime_residues(n))`
/// modulo `n`.
theorem product_mul_mod_residues_congr(n: Nat, a: Nat) {
    product[Nat](mul_mod_residues(n, a)).congr_mod(
        a.pow(n.totient) * product[Nat](coprime_residues(n)), n)
} by {
    product_map_mul_mod(n, a, coprime_residues(n))
    product[Nat](map(coprime_residues(n), mul_mod_fn(n, a))).congr_mod(
        a.pow(coprime_residues(n).length) * product[Nat](coprime_residues(n)), n)
    coprime_residues_length(n)
    coprime_residues(n).length = n.totient
    mul_mod_residues(n, a) = map(coprime_residues(n), mul_mod_fn(n, a))
    product[Nat](mul_mod_residues(n, a)) =
        product[Nat](map(coprime_residues(n), mul_mod_fn(n, a)))
}

/// `a^φ(n) * product(coprime_residues(n)) ≡ product(coprime_residues(n)) (mod n)`
/// for `n > 0` and `a` coprime to `n`. The permutation
/// `mul_mod_residues(n, a) ~ coprime_residues(n)` makes the two products
/// equal as naturals; combined with `product_mul_mod_residues_congr` this
/// gives the displayed congruence.
theorem euler_product_setup(n: Nat, a: Nat) {
    n != Nat.0 and a.coprime(n)
        implies (a.pow(n.totient) * product[Nat](coprime_residues(n))).congr_mod(
            product[Nat](coprime_residues(n)), n)
} by {
    if n != Nat.0 and a.coprime(n) {
        mul_mod_residues_is_permutation(n, a)
        is_permutation(mul_mod_residues(n, a), coprime_residues(n))
        permutation_preserves_product[Nat](mul_mod_residues(n, a), coprime_residues(n))
        product[Nat](mul_mod_residues(n, a)) = product[Nat](coprime_residues(n))
        product_mul_mod_residues_congr(n, a)
        product[Nat](mul_mod_residues(n, a)).congr_mod(a.pow(n.totient) * product[Nat](coprime_residues(n)), n)
        product[Nat](coprime_residues(n)).congr_mod(a.pow(n.totient) * product[Nat](coprime_residues(n)), n)
        congr_mod_symm(product[Nat](coprime_residues(n)), a.pow(n.totient) * product[Nat](coprime_residues(n)), n)
        (a.pow(n.totient) * product[Nat](coprime_residues(n))).congr_mod(product[Nat](coprime_residues(n)), n)
    }
}

/// Euler's theorem (general): if `a` is coprime to `n` and `n != 0`, then
/// `a^φ(n) ≡ 1 (mod n)`.
theorem euler(n: Nat, a: Nat) {
    n != Nat.0 and a.coprime(n)
        implies a.pow(n.totient).congr_mod(Nat.1, n)
} by {
    if n != Nat.0 and a.coprime(n) {
        euler_product_setup(n, a)
        (a.pow(n.totient) * product[Nat](coprime_residues(n))).congr_mod(product[Nat](coprime_residues(n)), n)
        // Reorder factors so that `cancel_coprime` can peel off
        // `product(coprime_residues(n))` on the left.
        a.pow(n.totient) * product[Nat](coprime_residues(n)) = product[Nat](coprime_residues(n)) * a.pow(n.totient)
        (product[Nat](coprime_residues(n)) * a.pow(n.totient)).congr_mod(product[Nat](coprime_residues(n)), n)
        product[Nat](coprime_residues(n)) * Nat.1 = product[Nat](coprime_residues(n))
        (product[Nat](coprime_residues(n)) * a.pow(n.totient)).congr_mod(product[Nat](coprime_residues(n)) * Nat.1, n)
        product_coprime_residues_coprime(n)
        product[Nat](coprime_residues(n)).coprime(n)
        cancel_coprime(product[Nat](coprime_residues(n)), n, a.pow(n.totient), Nat.1)
        a.pow(n.totient).congr_mod(Nat.1, n)
    }
}

/// `(q * m + r).mod(m) = r.mod(m)`. Adding any multiple of `m` to `r` does not
/// change the residue mod `m`. Reorientation of `mod_add_mul` to the `q * m`
/// form used in row-by-row counting on `[0, m * n)`.
theorem mod_add_mul_left(q: Nat, m: Nat, r: Nat) {
    (q * m + r).mod(m) = r.mod(m)
} by {
    mod_add_mul(q, m, r)
}

/// Coprimality on the `m`-side of the row decomposition `x = q * m + r`:
/// `(q * m + r).coprime(m) = r.coprime(m)`.
theorem coprime_add_mul_left(q: Nat, m: Nat, r: Nat) {
    (q * m + r).coprime(m) = r.coprime(m)
} by {
    coprime_mod_iff(q * m + r, m)
    (q * m + r).coprime(m) = (q * m + r).mod(m).coprime(m)
    mod_add_mul_left(q, m, r)
    (q * m + r).mod(m) = r.mod(m)
    (q * m + r).mod(m).coprime(m) = r.mod(m).coprime(m)
    coprime_mod_iff(r, m)
    r.coprime(m) = r.mod(m).coprime(m)
}


/// For `n != 0`, every `c` admits a `d` with `(c + d).mod(n) = 0`. Concretely
/// `d = n - c.mod(n)` so that `c + d` is the next multiple of `n`. Used to
/// turn additive cancellation modulo `n` into the existing `congr_mod_add`.
theorem mod_complement_exists(c: Nat, n: Nat) {
    n != Nat.0 implies exists(d: Nat) { (c + d).mod(n) = Nat.0 }
} by {
    if n != Nat.0 {
        mod_lt(c, n)
        c.mod(n) < n
        let d: Nat satisfy { d + c.mod(n) = n }
        add_mod(c, n)
        let q: Nat satisfy { q * n + c.mod(n) = c }
        c + d = q * n + c.mod(n) + d
        c.mod(n) + d = d + c.mod(n)
        c.mod(n) + d = n
        q * n + c.mod(n) + d = q * n + (c.mod(n) + d)
        q * n + (c.mod(n) + d) = q * n + n
        q * n + n = q * n + Nat.1 * n
        q * n + Nat.1 * n = (q + Nat.1) * n
        c + d = (q + Nat.1) * n
        (q + Nat.1) * n = (q + Nat.1) * n + Nat.0
        c + d = (q + Nat.1) * n + Nat.0
        mod_add_mul(q + Nat.1, n, Nat.0)
        ((q + Nat.1) * n + Nat.0).mod(n) = Nat.0.mod(n)
        Nat.0.mod(n) = Nat.0
        (c + d).mod(n) = Nat.0
        exists(e: Nat) { (c + e).mod(n) = Nat.0 }
    }
}

/// Helper: in a positive modulus, adding the same `c` to both sides cancels.
theorem congr_mod_add_cancel_right_pos(a: Nat, b: Nat, c: Nat, n: Nat) {
    n != Nat.0 and (a + c).congr_mod(b + c, n) implies a.congr_mod(b, n)
} by {
    if n != Nat.0 and (a + c).congr_mod(b + c, n) {
        mod_complement_exists(c, n)
        let d: Nat satisfy { (c + d).mod(n) = Nat.0 }
        (c + d).mod(n) = Nat.0
        // (a+c+d) ≡ (b+c+d) by adding d to both sides.
        congr_mod_refl(d, n)
        congr_mod_add(a + c, d, b + c, d, n)
        ((a + c) + d).congr_mod((b + c) + d, n)
        (a + c) + d = a + (c + d)
        (b + c) + d = b + (c + d)
        (a + (c + d)).congr_mod(b + (c + d), n)
        // Reduce a + (c + d) ≡ a (mod n).
        mod_add_eq(a, c + d, n)
        (a + (c + d)).mod(n) = (a.mod(n) + (c + d).mod(n)).mod(n)
        a.mod(n) + (c + d).mod(n) = a.mod(n) + Nat.0
        a.mod(n) + Nat.0 = a.mod(n)
        (a.mod(n) + (c + d).mod(n)).mod(n) = a.mod(n).mod(n)
        mod_mod(a, n)
        a.mod(n).mod(n) = a.mod(n)
        (a + (c + d)).mod(n) = a.mod(n)
        // Same for b.
        mod_add_eq(b, c + d, n)
        (b + (c + d)).mod(n) = (b.mod(n) + (c + d).mod(n)).mod(n)
        b.mod(n) + (c + d).mod(n) = b.mod(n) + Nat.0
        b.mod(n) + Nat.0 = b.mod(n)
        (b.mod(n) + (c + d).mod(n)).mod(n) = b.mod(n).mod(n)
        mod_mod(b, n)
        b.mod(n).mod(n) = b.mod(n)
        (b + (c + d)).mod(n) = b.mod(n)
        (a + (c + d)).mod(n) = (b + (c + d)).mod(n)
        a.mod(n) = b.mod(n)
    }
}

/// Helper: in modulus zero, congr_mod is just equality, and addition cancels.
theorem congr_mod_add_cancel_right_zero(a: Nat, b: Nat, c: Nat) {
    (a + c).congr_mod(b + c, Nat.0) implies a.congr_mod(b, Nat.0)
} by {
    if (a + c).congr_mod(b + c, Nat.0) {
        mod_by_zero(a + c)
        mod_by_zero(b + c)
        (a + c).mod(Nat.0) = a + c
        (b + c).mod(Nat.0) = b + c
        a + c = b + c
        c + a = a + c
        c + b = b + c
        c + a = c + b
        add_cancels_left(c, a, b)
        a = b
        mod_by_zero(a)
        mod_by_zero(b)
        a.mod(Nat.0) = a
        b.mod(Nat.0) = b
        a.mod(Nat.0) = b.mod(Nat.0)
    }
}

/// Injectivity of the affine map `q -> (q * m + r).mod(n)` on `[0, n)`
/// whenever `m.coprime(n)`. Building block for the row-by-row Euler totient
/// argument: across each row `r` of `[0, m * n)`, this map permutes residues
/// in `[0, n)`, so each row contributes the same `n.totient` count.
theorem add_mul_mod_inj_below(m: Nat, n: Nat, r: Nat, x: Nat, y: Nat) {
    n != Nat.0 and m.coprime(n) and x < n and y < n
        and (x * m + r).mod(n) = (y * m + r).mod(n)
        implies x = y
} by {
    if n != Nat.0 and m.coprime(n) and x < n and y < n
        and (x * m + r).mod(n) = (y * m + r).mod(n) {
        // Cancel the additive offset `r` from the congruence.
        (x * m + r).congr_mod(y * m + r, n)
        congr_mod_add_cancel_right_pos(x * m, y * m, r, n)
        (x * m).congr_mod(y * m, n)
        (x * m).mod(n) = (y * m).mod(n)
        // Reorient as `m * x` and apply the existing injection.
        x * m = m * x
        y * m = m * y
        (m * x).mod(n) = (m * y).mod(n)
        mul_mod_inj_below(n, m, x, y)
        x = y
    }
}

/// Top-level affine point function for full residue rows: `q ↦ (q*m+r) mod n`.
/// Naming the function keeps map-based statements stable under certificate
/// replay, matching the existing `mul_mod_fn` convention for unit rows.
define add_mul_mod_fn(n: Nat, m: Nat, r: Nat) -> (Nat -> Nat) {
    function(q: Nat) { (q * m + r).mod(n) }
}

/// The affine image of the full residue range `[0, n)`.
define add_mul_mod_residues(n: Nat, m: Nat, r: Nat) -> List[Nat] {
    map(n.range, add_mul_mod_fn(n, m, r))
}

/// Coprimality of a row entry with the product modulus splits into the row
/// residue's `m`-side coprimality and the entry's `n`-side coprimality.
theorem row_entry_coprime_product_iff(m: Nat, n: Nat, q: Nat, r: Nat) {
    (q * m + r).coprime(m * n) =
        (r.coprime(m) and (q * m + r).coprime(n))
} by {
    if (q * m + r).coprime(m * n) {
        coprime_mul_iff(q * m + r, m, n)
        (q * m + r).coprime(m)
        (q * m + r).coprime(n)
        coprime_add_mul_left(q, m, r)
        (q * m + r).coprime(m) = r.coprime(m)
        r.coprime(m)
        r.coprime(m) and (q * m + r).coprime(n)
    }
    if r.coprime(m) and (q * m + r).coprime(n) {
        coprime_add_mul_left(q, m, r)
        (q * m + r).coprime(m) = r.coprime(m)
        (q * m + r).coprime(m)
        coprime_mul(q * m + r, m, n)
        (q * m + r).coprime(m * n)
    }
}

/// Product-modulus row-entry coprimality, expressed through the normalized row
/// value used by `add_mul_mod_units`.
theorem row_entry_coprime_product_mod_iff(m: Nat, n: Nat, q: Nat, r: Nat) {
    (q * m + r).coprime(m * n) =
        (r.coprime(m) and add_mul_mod_fn(n, m, r)(q).coprime(n))
} by {
    row_entry_coprime_product_iff(m, n, q, r)
    (q * m + r).coprime(m * n) =
        (r.coprime(m) and (q * m + r).coprime(n))
    coprime_mod_iff(q * m + r, n)
    (q * m + r).coprime(n) = (q * m + r).mod(n).coprime(n)
    add_mul_mod_fn(n, m, r)(q) = (q * m + r).mod(n)
    (q * m + r).coprime(n) = add_mul_mod_fn(n, m, r)(q).coprime(n)
    (r.coprime(m) and (q * m + r).coprime(n)) =
        (r.coprime(m) and add_mul_mod_fn(n, m, r)(q).coprime(n))
    (q * m + r).coprime(m * n) =
        (r.coprime(m) and add_mul_mod_fn(n, m, r)(q).coprime(n))
}

/// The affine row map always lands below a positive modulus.
theorem add_mul_mod_fn_below(n: Nat, m: Nat, r: Nat, q: Nat) {
    n != Nat.0 implies add_mul_mod_fn(n, m, r)(q) < n
} by {
    if n != Nat.0 {
        add_mul_mod_fn(n, m, r)(q) = (q * m + r).mod(n)
        mod_lt(q * m + r, n)
        (q * m + r).mod(n) < n
    }
}

/// Surjectivity below `n` for the affine full-residue row
/// `q ↦ (q*m+r) mod n`, when `m` is a unit modulo `n`.
theorem add_mul_mod_surj_below(n: Nat, m: Nat, r: Nat, s: Nat) {
    n != Nat.0 and m.coprime(n) and s < n implies exists(q: Nat) {
        q < n and (q * m + r).mod(n) = s
    }
} by {
    if n != Nat.0 and m.coprime(n) and s < n {
        // Choose an additive complement `d` with r + d ≡ 0, then solve
        // q*m ≡ s+d using q = (m^{-1}*(s+d)) mod n.
        mod_complement_exists(r, n)
        let d: Nat satisfy { (r + d).mod(n) = Nat.0 }
        let b: Nat = mod_inv(m, n)
        let q: Nat = (b * (s + d)).mod(n)
        mod_lt(b * (s + d), n)
        q < n

        // q*m is congruent to s+d.
        mod_congr_mod_self(b * (s + d), n)
        q.congr_mod(b * (s + d), n)
        congr_mod_refl(m, n)
        m.congr_mod(m, n)
        congr_mod_mul(q, m, b * (s + d), m, n)
        (q * m).congr_mod((b * (s + d)) * m, n)
        (b * (s + d)) * m = (m * b) * (s + d)
        mod_inv_mul_congr_one(m, n)
        (m * b).congr_mod(Nat.1, n)
        scale_inv_by_x(m, b, s + d, n)
        ((m * b) * (s + d)).congr_mod(s + d, n)
        congr_mod_trans(q * m, (b * (s + d)) * m, s + d, n)
        (q * m).congr_mod(s + d, n)

        // Add r to both sides, then use r+d ≡ 0 to collapse the right side to s.
        congr_mod_refl(r, n)
        r.congr_mod(r, n)
        congr_mod_add(q * m, r, s + d, r, n)
        (q * m + r).congr_mod((s + d) + r, n)
        (s + d) + r = s + (d + r)
        d + r = r + d
        (s + d) + r = s + (r + d)
        mod_add_eq(s, r + d, n)
        (s + (r + d)).mod(n) = (s.mod(n) + (r + d).mod(n)).mod(n)
        small_mod(s, n)
        s.mod(n) = s
        (r + d).mod(n) = Nat.0
        s.mod(n) + (r + d).mod(n) = s + Nat.0
        s + Nat.0 = s
        (s.mod(n) + (r + d).mod(n)).mod(n) = s.mod(n)
        (s + (r + d)).mod(n) = s.mod(n)
        (s + (r + d)).mod(n) = s
        (q * m + r).mod(n) = ((s + d) + r).mod(n)
        ((s + d) + r).mod(n) = (s + (r + d)).mod(n)
        (q * m + r).mod(n) = s
        q < n and (q * m + r).mod(n) = s
        exists(q0: Nat) { q0 < n and (q0 * m + r).mod(n) = s }
    }
}

/// Every element of the affine image list is below `n`.
theorem add_mul_mod_residues_contains_imp_below(n: Nat, m: Nat, r: Nat, s: Nat) {
    n != Nat.0 and add_mul_mod_residues(n, m, r).contains(s) implies s < n
} by {
    if n != Nat.0 and add_mul_mod_residues(n, m, r).contains(s) {
        map(n.range, add_mul_mod_fn(n, m, r)).contains(s)
        map_contains(n.range, add_mul_mod_fn(n, m, r), s)
        let q: Nat satisfy {
            n.range.contains(q) and add_mul_mod_fn(n, m, r)(q) = s
        }
        add_mul_mod_fn_below(n, m, r, q)
        add_mul_mod_fn(n, m, r)(q) < n
        s < n
    }
}

/// Every `s < n` occurs in the affine image list.
theorem add_mul_mod_residues_contains_intro(n: Nat, m: Nat, r: Nat, s: Nat) {
    n != Nat.0 and m.coprime(n) and s < n
        implies add_mul_mod_residues(n, m, r).contains(s)
} by {
    if n != Nat.0 and m.coprime(n) and s < n {
        add_mul_mod_surj_below(n, m, r, s)
        let q: Nat satisfy { q < n and (q * m + r).mod(n) = s }
        range_contains_iff_lt(n, q)
        n.range.contains(q) = (q < n)
        n.range.contains(q)
        add_mul_mod_fn(n, m, r)(q) = (q * m + r).mod(n)
        add_mul_mod_fn(n, m, r)(q) = s
        map_contains_of_contains(n.range, add_mul_mod_fn(n, m, r), q)
        map(n.range, add_mul_mod_fn(n, m, r)).contains(add_mul_mod_fn(n, m, r)(q))
        map(n.range, add_mul_mod_fn(n, m, r)).contains(s)
    }
}

/// Forward half of the affine full-residue membership characterization.
lemma add_mul_mod_residues_contains_only_below(n: Nat, m: Nat, r: Nat, s: Nat) {
    n != Nat.0 and m.coprime(n) and add_mul_mod_residues(n, m, r).contains(s)
        implies s < n
} by {
    if n != Nat.0 and m.coprime(n) and add_mul_mod_residues(n, m, r).contains(s) {
        add_mul_mod_residues_contains_imp_below(n, m, r, s)
    }
}

/// Backward half of the affine full-residue membership characterization.
lemma add_mul_mod_residues_contains_all_below(n: Nat, m: Nat, r: Nat, s: Nat) {
    n != Nat.0 and m.coprime(n) and s < n
        implies add_mul_mod_residues(n, m, r).contains(s)
} by {
    if n != Nat.0 and m.coprime(n) and s < n {
        add_mul_mod_residues_contains_intro(n, m, r, s)
    }
}

/// Membership characterization for the affine full-residue image: under a
/// positive modulus and unit stride, its elements are exactly `[0, n)`.
theorem add_mul_mod_residues_contains_iff(n: Nat, m: Nat, r: Nat, s: Nat) {
    n != Nat.0 and m.coprime(n)
        implies add_mul_mod_residues(n, m, r).contains(s) = (s < n)
} by {
    if n != Nat.0 and m.coprime(n) {
        if add_mul_mod_residues(n, m, r).contains(s) {
            add_mul_mod_residues_contains_only_below(n, m, r, s)
            s < n
            add_mul_mod_residues(n, m, r).contains(s) = true
            (s < n) = true
            add_mul_mod_residues(n, m, r).contains(s) = (s < n)
        }
        if not add_mul_mod_residues(n, m, r).contains(s) {
            if s < n {
                add_mul_mod_residues_contains_all_below(n, m, r, s)
                add_mul_mod_residues(n, m, r).contains(s)
                false
            }
            not (s < n)
            add_mul_mod_residues(n, m, r).contains(s) = false
            (s < n) = false
            add_mul_mod_residues(n, m, r).contains(s) = (s < n)
        }
    }
}

/// The affine row list has length `n`, because it maps over `n.range`.
theorem add_mul_mod_residues_length(n: Nat, m: Nat, r: Nat) {
    add_mul_mod_residues(n, m, r).length = n
} by {
    map_length(n.range, add_mul_mod_fn(n, m, r))
    map(n.range, add_mul_mod_fn(n, m, r)).length = n.range.length
    length_range(n)
    n.range.length = n
}

/// Inductive step for local range uniqueness: appending a fresh maximum
/// preserves uniqueness.
lemma add_mul_nat_range_unique_suc(k: Nat) {
    k.range.is_unique implies k.suc.range.is_unique
} by {
    if k.range.is_unique {
        singleton_unique(k)
        List.singleton(k).is_unique
        range_does_not_contain_geq(k, k)
        k >= k
        not k.range.contains(k)
        List.singleton(k) = List.cons(k, List.nil[Nat])
        forall(y: Nat) {
            if List.singleton(k).contains(y) {
                List.cons(k, List.nil[Nat]).contains(y)
                not List.nil[Nat].contains(y)
                y = k
                not k.range.contains(y)
            }
            not (k.range.contains(y) and List.singleton(k).contains(y))
        }
        unique_list_sum(k.range, List.singleton(k))
        (k.range + List.singleton(k)).is_unique
        k.suc.range = k.range.append(k)
        k.range.append(k) = k.range + List.singleton(k)
        k.suc.range.is_unique
    }
}

/// Local copy of range uniqueness used by the affine row package.
lemma add_mul_nat_range_unique(n: Nat) {
    n.range.is_unique
} by {
    let p: Nat -> Bool = function(k: Nat) {
        k.range.is_unique
    }
    Nat.0.range = List.nil[Nat]
    List.nil[Nat].unique = List.nil[Nat]
    List.nil[Nat].is_unique
    p(Nat.0)
    forall(k: Nat) {
        if p(k) {
            add_mul_nat_range_unique_suc(k)
            p(k.suc)
        }
    }
    forall(k: Nat) { p(k) implies p(k.suc) }
    p(Nat.0) and forall(k: Nat) { p(k) implies p(k.suc) }
    alt_induction(p)
    forall(k: Nat) { p(k) }
    p(n)
}

/// In a unique Nat cons-list, the head is absent from the tail. Local copy of
/// the count proof used by finite_fiber_partition; avoids editing list files.
lemma nat_unique_cons_not_contains_by_count(head: Nat, tail: List[Nat]) {
    List.cons(head, tail).is_unique implies not tail.contains(head)
} by {
    if tail.contains(head) {
        list_contains_implies_count_geq_one[Nat](tail, head)
        tail.count(head) >= Nat.1
        List.cons(head, tail).count(head) =
            (if head = head { Nat.1 + tail.count(head) } else { tail.count(head) })
        List.cons(head, tail).count(head) = Nat.1 + tail.count(head)
        sum_lte(Nat.1, Nat.1, Nat.1, tail.count(head))
        Nat.1 + Nat.1 <= Nat.1 + tail.count(head)
        Nat.1 + Nat.1 = Nat.1.suc
        Nat.1.suc <= Nat.1 + tail.count(head)
        List.cons(head, tail).count(head) >= Nat.1.suc
        unique_implies_no_duplicate[Nat](List.cons(head, tail), head)
        List.cons(head, tail).count(head) <= Nat.1
        lte_trans(Nat.1.suc, List.cons(head, tail).count(head), Nat.1)
        Nat.1.suc <= Nat.1
        false
    }
}

/// Filtering a unique Nat-list preserves uniqueness. Kept local because this
/// lane does not edit list infrastructure.
lemma nat_filter_preserves_unique(list: List[Nat], f: Nat -> Bool) {
    list.is_unique implies list.filter(f).is_unique
} by {
    define p(xs: List[Nat]) -> Bool {
        xs.is_unique implies xs.filter(f).is_unique
    }
    List.nil[Nat].filter(f) = List.nil[Nat]
    List.nil[Nat].unique = List.nil[Nat]
    List.nil[Nat].is_unique
    List.nil[Nat].filter(f).unique = List.nil[Nat]
    List.nil[Nat].filter(f).is_unique
    p(List.nil[Nat]) = (List.nil[Nat].is_unique implies List.nil[Nat].filter(f).is_unique)
    List.nil[Nat].is_unique implies List.nil[Nat].filter(f).is_unique
    p(List.nil[Nat])
    forall(head: Nat, tail: List[Nat]) {
        if p(tail) {
            if List.cons(head, tail).is_unique {
                unique_implies_tail_unique(head, tail)
                nat_unique_cons_not_contains_by_count(head, tail)
                tail.is_unique
                p(tail) = (tail.is_unique implies tail.filter(f).is_unique)
                tail.is_unique implies tail.filter(f).is_unique
                tail.filter(f).is_unique
                not tail.contains(head)
                if f(head) {
                    List.cons(head, tail).filter(f) = List.cons(head, tail.filter(f))
                    if tail.filter(f).contains(head) {
                        filter_contained_by_and(tail, f, head)
                        tail.contains(head)
                        false
                    }
                    not tail.filter(f).contains(head)
                    cons_unique_intro(head, tail.filter(f))
                    List.cons(head, tail.filter(f)).is_unique
                    List.cons(head, tail).filter(f).is_unique
                }
                if not f(head) {
                    List.cons(head, tail).filter(f) =
                        (if f(head) { List.cons(head, tail.filter(f)) } else { tail.filter(f) })
                    List.cons(head, tail).filter(f) = tail.filter(f)
                    List.cons(head, tail).filter(f).is_unique
                }
                List.cons(head, tail).filter(f).is_unique
            }
            if not List.cons(head, tail).is_unique {
            }
            p(List.cons(head, tail)) = (List.cons(head, tail).is_unique implies List.cons(head, tail).filter(f).is_unique)
            p(List.cons(head, tail))
        }
    }
    forall(head: Nat, tail: List[Nat]) {
        p(tail) implies p(List.cons(head, tail))
    }
    p(List.nil[Nat]) and forall(head: Nat, tail: List[Nat]) {
        p(tail) implies p(List.cons(head, tail))
    }
    List.induction(function(xs: List[Nat]) { p(xs) })
    forall(xs: List[Nat]) { p(xs) }
    p(list)
}

/// The filtered natural range contains exactly the residues below `n` that are
/// coprime to `n`.
theorem range_filter_coprime_contains_iff(n: Nat, x: Nat) {
    n.range.filter(function(y: Nat) { y.coprime(n) }).contains(x) =
        (x < n and x.coprime(n))
} by {
    filter_equivalent_to_and(n.range, function(y: Nat) { y.coprime(n) }, x)
    (n.range.contains(x) and x.coprime(n)) =
        n.range.filter(function(y: Nat) { y.coprime(n) }).contains(x)
    range_contains_iff_lt(n, x)
    n.range.contains(x) = (x < n)
    (n.range.contains(x) and x.coprime(n)) = (x < n and x.coprime(n))
    n.range.filter(function(y: Nat) { y.coprime(n) }).contains(x) =
        (x < n and x.coprime(n))
}

/// The filtered natural range is a duplicate-free list.
lemma range_filter_coprime_unique(n: Nat) {
    n.range.filter(function(x: Nat) { x.coprime(n) }).is_unique
} by {
    add_mul_nat_range_unique(n)
    n.range.is_unique
    nat_filter_preserves_unique(n.range, function(x: Nat) { x.coprime(n) })
}

/// Filtering `n.range` by coprimality has Euler-totient length.
theorem range_filter_coprime_length(n: Nat) {
    n.range.filter(function(x: Nat) { x.coprime(n) }).length = n.totient
} by {
    let filtered: List[Nat] = n.range.filter(function(x: Nat) { x.coprime(n) })
    range_filter_coprime_unique(n)
    filtered.is_unique
    coprime_residues_unique(n)
    coprime_residues(n).is_unique
    forall(x: Nat) {
        range_filter_coprime_contains_iff(n, x)
        filtered.contains(x) = (x < n and x.coprime(n))
        coprime_residues_contains_iff(n, x)
        coprime_residues(n).contains(x) = (x < n and x.coprime(n))
        filtered.contains(x) = coprime_residues(n).contains(x)
    }
    unique_same_contains_imp_permutation(filtered, coprime_residues(n))
    is_permutation(filtered, coprime_residues(n))
    permutation_preserves_length(filtered, coprime_residues(n))
    filtered.length = coprime_residues(n).length
    coprime_residues_length(n)
    coprime_residues(n).length = n.totient
    filtered.length = n.totient
}

/// The affine row function is injective on `n.range` when the stride is a unit
/// modulo the positive modulus `n`.
theorem add_mul_mod_fn_locally_injective_on_range(n: Nat, m: Nat, r: Nat) {
    n != Nat.0 and m.coprime(n) implies forall(x: Nat, y: Nat) {
        n.range.contains(x) and n.range.contains(y) and
            add_mul_mod_fn(n, m, r)(x) = add_mul_mod_fn(n, m, r)(y)
            implies x = y
    }
} by {
    if n != Nat.0 and m.coprime(n) {
        forall(x: Nat, y: Nat) {
            if n.range.contains(x) and n.range.contains(y) and
                add_mul_mod_fn(n, m, r)(x) = add_mul_mod_fn(n, m, r)(y) {
                range_contains_iff_lt(n, x)
                n.range.contains(x) = (x < n)
                x < n
                range_contains_iff_lt(n, y)
                n.range.contains(y) = (y < n)
                y < n
                add_mul_mod_fn(n, m, r)(x) = (x * m + r).mod(n)
                add_mul_mod_fn(n, m, r)(y) = (y * m + r).mod(n)
                (x * m + r).mod(n) = (y * m + r).mod(n)
                add_mul_mod_inj_below(m, n, r, x, y)
                x = y
            }
        }
    }
}

/// The affine full-residue row has no duplicates when the stride is a unit.
theorem add_mul_mod_residues_unique(n: Nat, m: Nat, r: Nat) {
    n != Nat.0 and m.coprime(n) implies add_mul_mod_residues(n, m, r).is_unique
} by {
    if n != Nat.0 and m.coprime(n) {
        add_mul_nat_range_unique(n)
        n.range.is_unique
        add_mul_mod_fn_locally_injective_on_range(n, m, r)
        locally_injective_map_is_unique(n.range, add_mul_mod_fn(n, m, r))
        map(n.range, add_mul_mod_fn(n, m, r)).is_unique
        add_mul_mod_residues(n, m, r) = map(n.range, add_mul_mod_fn(n, m, r))
        add_mul_mod_residues(n, m, r).is_unique
    }
}

/// The affine full-residue row and the natural range have the same membership
/// predicate.
theorem add_mul_mod_residues_same_contains_range(n: Nat, m: Nat, r: Nat) {
    n != Nat.0 and m.coprime(n) implies forall(x: Nat) {
        add_mul_mod_residues(n, m, r).contains(x) = n.range.contains(x)
    }
} by {
    if n != Nat.0 and m.coprime(n) {
        forall(x: Nat) {
            add_mul_mod_residues_contains_iff(n, m, r, x)
            add_mul_mod_residues(n, m, r).contains(x) = (x < n)
            range_contains_iff_lt(n, x)
            n.range.contains(x) = (x < n)
            add_mul_mod_residues(n, m, r).contains(x) = n.range.contains(x)
        }
    }
}

/// The affine full-residue row is a permutation of the natural range `[0, n)`.
theorem add_mul_mod_residues_is_permutation(n: Nat, m: Nat, r: Nat) {
    n != Nat.0 and m.coprime(n) implies is_permutation(add_mul_mod_residues(n, m, r), n.range)
} by {
    if n != Nat.0 and m.coprime(n) {
        add_mul_mod_residues_unique(n, m, r)
        add_mul_mod_residues(n, m, r).is_unique
        add_mul_nat_range_unique(n)
        n.range.is_unique
        add_mul_mod_residues_same_contains_range(n, m, r)
        forall(x: Nat) {
            add_mul_mod_residues(n, m, r).contains(x) = n.range.contains(x)
        }
        unique_same_contains_imp_permutation(add_mul_mod_residues(n, m, r), n.range)
        is_permutation(add_mul_mod_residues(n, m, r), n.range)
    }
}

/// Units in an affine row: filter the full row image by coprimality to `n`.
define add_mul_mod_units(n: Nat, m: Nat, r: Nat) -> List[Nat] {
    add_mul_mod_residues(n, m, r).filter(function(x: Nat) { x.coprime(n) })
}

/// The affine row units and the coprime part of `n.range` have the same
/// membership predicate.
theorem add_mul_mod_units_same_contains_range_filter(n: Nat, m: Nat, r: Nat) {
    n != Nat.0 and m.coprime(n) implies forall(x: Nat) {
        add_mul_mod_units(n, m, r).contains(x) =
            n.range.filter(function(y: Nat) { y.coprime(n) }).contains(x)
    }
} by {
    if n != Nat.0 and m.coprime(n) {
        forall(x: Nat) {
            filter_equivalent_to_and(add_mul_mod_residues(n, m, r), function(y: Nat) { y.coprime(n) }, x)
            (add_mul_mod_residues(n, m, r).contains(x) and x.coprime(n)) = add_mul_mod_units(n, m, r).contains(x)
            add_mul_mod_residues_contains_iff(n, m, r, x)
            add_mul_mod_residues(n, m, r).contains(x) = (x < n)
            (add_mul_mod_residues(n, m, r).contains(x) and x.coprime(n)) = (x < n and x.coprime(n))
            add_mul_mod_units(n, m, r).contains(x) = (x < n and x.coprime(n))
            range_filter_coprime_contains_iff(n, x)
            n.range.filter(function(y: Nat) { y.coprime(n) }).contains(x) = (x < n and x.coprime(n))
            add_mul_mod_units(n, m, r).contains(x) = n.range.filter(function(y: Nat) { y.coprime(n) }).contains(x)
        }
    }
}

/// The affine row units are the same finite set as the coprime part of `n.range`.
theorem add_mul_mod_units_is_permutation_range_filter(n: Nat, m: Nat, r: Nat) {
    n != Nat.0 and m.coprime(n) implies
        is_permutation(add_mul_mod_units(n, m, r),
            n.range.filter(function(x: Nat) { x.coprime(n) }))
} by {
    if n != Nat.0 and m.coprime(n) {
        add_mul_mod_residues_unique(n, m, r)
        add_mul_mod_residues(n, m, r).is_unique
        nat_filter_preserves_unique(add_mul_mod_residues(n, m, r), function(x: Nat) { x.coprime(n) })
        add_mul_mod_units(n, m, r).is_unique
        range_filter_coprime_unique(n)
        n.range.filter(function(x: Nat) { x.coprime(n) }).is_unique
        add_mul_mod_units_same_contains_range_filter(n, m, r)
        forall(x: Nat) {
            add_mul_mod_units(n, m, r).contains(x) = n.range.filter(function(y: Nat) { y.coprime(n) }).contains(x)
        }
        unique_same_contains_imp_permutation(add_mul_mod_units(n, m, r),
            n.range.filter(function(x: Nat) { x.coprime(n) }))
        is_permutation(add_mul_mod_units(n, m, r),
            n.range.filter(function(x: Nat) { x.coprime(n) }))
    }
}

/// Each affine row has exactly `n.totient` unit residues.
theorem add_mul_mod_units_length(n: Nat, m: Nat, r: Nat) {
    n != Nat.0 and m.coprime(n) implies add_mul_mod_units(n, m, r).length = n.totient
} by {
    if n != Nat.0 and m.coprime(n) {
        add_mul_mod_units_is_permutation_range_filter(n, m, r)
        is_permutation(add_mul_mod_units(n, m, r), n.range.filter(function(x: Nat) { x.coprime(n) }))
        permutation_preserves_length(add_mul_mod_units(n, m, r), n.range.filter(function(x: Nat) { x.coprime(n) }))
        add_mul_mod_units(n, m, r).length = n.range.filter(function(x: Nat) { x.coprime(n) }).length
        range_filter_coprime_length(n)
        n.range.filter(function(x: Nat) { x.coprime(n) }).length = n.totient
        add_mul_mod_units(n, m, r).length = n.totient
    }
}

/// When the fixed row residue is coprime to `m`, the row entries coprime to
/// `m*n` are counted by the unit residues modulo `n`.
theorem row_coprime_product_filter_length_if_coprime(m: Nat, n: Nat, r: Nat) {
    n != Nat.0 and m.coprime(n) and r.coprime(m) implies
        n.range.filter(function(q: Nat) { (q * m + r).coprime(m * n) }).length = n.totient
} by {
    if n != Nat.0 and m.coprime(n) and r.coprime(m) {
        forall(q: Nat) {
            if n.range.contains(q) {
                row_entry_coprime_product_mod_iff(m, n, q, r)
                (q * m + r).coprime(m * n) =
                    (r.coprime(m) and add_mul_mod_fn(n, m, r)(q).coprime(n))
                (r.coprime(m) and add_mul_mod_fn(n, m, r)(q).coprime(n)) =
                    add_mul_mod_fn(n, m, r)(q).coprime(n)
                (q * m + r).coprime(m * n) = add_mul_mod_fn(n, m, r)(q).coprime(n)
            }
        }
        map_filter_length_of_pointwise[Nat, Nat](
            n.range,
            add_mul_mod_fn(n, m, r),
            function(x: Nat) { x.coprime(n) },
            function(q: Nat) { (q * m + r).coprime(m * n) }
        )
        map[Nat, Nat](n.range, add_mul_mod_fn(n, m, r)).filter(function(x: Nat) { x.coprime(n) }).length =
            n.range.filter(function(q: Nat) { (q * m + r).coprime(m * n) }).length
        add_mul_mod_residues(n, m, r) = map(n.range, add_mul_mod_fn(n, m, r))
        add_mul_mod_units(n, m, r) = add_mul_mod_residues(n, m, r).filter(function(x: Nat) { x.coprime(n) })
        add_mul_mod_units(n, m, r).length =
            n.range.filter(function(q: Nat) { (q * m + r).coprime(m * n) }).length
        add_mul_mod_units_length(n, m, r)
        add_mul_mod_units(n, m, r).length = n.totient
        n.range.filter(function(q: Nat) { (q * m + r).coprime(m * n) }).length = n.totient
    }
}

/// A Nat-list with no members has zero length. This local bridge avoids
/// introducing any new public list API in the totient row-count lane.
lemma nat_no_contains_length_zero(items: List[Nat]) {
    forall(x: Nat) { not items.contains(x) } implies items.length = Nat.0
} by {
    define p(xs: List[Nat]) -> Bool {
        forall(x: Nat) { not xs.contains(x) } implies xs.length = Nat.0
    }
    List.nil[Nat].length = Nat.0
    p(List.nil[Nat])
    forall(head: Nat, tail: List[Nat]) {
        if p(tail) {
            if forall(x: Nat) { not List.cons[Nat](head, tail).contains(x) } {
                List.cons[Nat](head, tail).contains(head)
                not List.cons[Nat](head, tail).contains(head)
                false
            }
            p(List.cons[Nat](head, tail))
        }
    }
    forall(head: Nat, tail: List[Nat]) {
        p(tail) implies p(List.cons[Nat](head, tail))
    }
    p(List.nil[Nat]) and forall(head: Nat, tail: List[Nat]) {
        p(tail) implies p(List.cons[Nat](head, tail))
    }
    List.induction(function(xs: List[Nat]) { p(xs) })
    forall(xs: List[Nat]) { p(xs) }
    p(items)
}

/// If a Nat-list predicate is pointwise false on all elements of the list, the
/// corresponding filter has zero length. Kept local to the totient row-count
/// lane; the generic `filter_false_length` helper is private to `list`, and this
/// specialized bridge avoids editing list APIs.
lemma nat_filter_false_length_of_pointwise(items: List[Nat], f: Nat -> Bool) {
    forall(x: Nat) { items.contains(x) implies not f(x) }
        implies items.filter(f).length = Nat.0
} by {
    if forall(x: Nat) { items.contains(x) implies not f(x) } {
        forall(x: Nat) {
            if items.filter(f).contains(x) {
                filter_contained_by_and(items, f, x)
                items.contains(x) and f(x)
                items.contains(x)
                f(x)
                not f(x)
                false
            }
            not items.filter(f).contains(x)
        }
        nat_no_contains_length_zero(items.filter(f))
        items.filter(f).length = Nat.0
    }
}

/// When the fixed row residue is not coprime to `m`, no row entry is coprime
/// to the product modulus: every entry has the same `m`-side coprimality as `r`.
theorem row_coprime_product_filter_length_if_not_coprime(m: Nat, n: Nat, r: Nat) {
    not r.coprime(m) implies
        n.range.filter(function(q: Nat) { (q * m + r).coprime(m * n) }).length = Nat.0
} by {
    if not r.coprime(m) {
        forall(q: Nat) {
            if n.range.contains(q) {
                if (q * m + r).coprime(m * n) {
                    coprime_mul_iff(q * m + r, m, n)
                    (q * m + r).coprime(m)
                    coprime_add_mul_left(q, m, r)
                    (q * m + r).coprime(m) = r.coprime(m)
                    r.coprime(m)
                    false
                }
                not (q * m + r).coprime(m * n)
                (q * m + r).coprime(m * n) = false
            }
        }
        nat_filter_false_length_of_pointwise(n.range,
            function(q: Nat) { (q * m + r).coprime(m * n) })
        n.range.filter(function(q: Nat) { (q * m + r).coprime(m * n) }).length = Nat.0
    }
}

/// True when a natural number is coprime to a fixed modulus.
define coprime_to_pred(n: Nat) -> (Nat -> Bool) {
    function(x: Nat) { x.coprime(n) }
}

/// The product-modulus coprimality predicate.
define totient_product_pred(m: Nat, n: Nat) -> (Nat -> Bool) {
    coprime_to_pred(m * n)
}

/// The rectangle predicate induced by product-modulus coprimality.
define totient_product_pair_pred(m: Nat, n: Nat) -> (Pair[Nat, Nat] -> Bool) {
    row_encode_filter_pred(m, totient_product_pred(m, n))
}

/// The row predicate for entries `q * m + r` in the product rectangle.
define totient_row_pred(m: Nat, n: Nat, r: Nat) -> (Nat -> Bool) {
    function(q: Nat) { (q * m + r).coprime(m * n) }
}

/// The row contribution: `n.totient` on rows coprime to `m`, and zero
/// otherwise.
define totient_row_contribution(m: Nat, n: Nat) -> (Nat -> Nat) {
    filter_indicator_value(n.totient, coprime_to_pred(m))
}

/// The row predicate induced from `row_encode` is the direct row predicate.
lemma totient_product_pair_row_pred(m: Nat, n: Nat, r: Nat) {
    function(q: Nat) { totient_product_pair_pred(m, n)(Pair.new(r, q)) } =
        totient_row_pred(m, n, r)
} by {
    forall(q: Nat) {
        totient_product_pair_pred(m, n)(Pair.new(r, q)) =
            row_encode_filter_pred(m, totient_product_pred(m, n))(Pair.new(r, q))
        row_encode_filter_pred(m, totient_product_pred(m, n))(Pair.new(r, q)) =
            totient_product_pred(m, n)(row_encode(m)(Pair.new(r, q)))
        row_encode(m)(Pair.new(r, q)) =
            Pair.new(r, q).second * m + Pair.new(r, q).first
        Pair.new(r, q).second = q
        Pair.new(r, q).first = r
        row_encode(m)(Pair.new(r, q)) = q * m + r
        totient_product_pred(m, n)(row_encode(m)(Pair.new(r, q))) =
            (q * m + r).coprime(m * n)
        totient_row_pred(m, n, r)(q) = (q * m + r).coprime(m * n)
        totient_product_pair_pred(m, n)(Pair.new(r, q)) =
            totient_row_pred(m, n, r)(q)
    }
}

/// A coprime row contributes `n.totient` selected entries.
lemma totient_row_pred_filter_length_if_coprime(m: Nat, n: Nat, r: Nat) {
    n != Nat.0 and m.coprime(n) and r.coprime(m) implies
        n.range.filter(totient_row_pred(m, n, r)).length = n.totient
} by {
    if n != Nat.0 and m.coprime(n) and r.coprime(m) {
        row_coprime_product_filter_length_if_coprime(m, n, r)
        n.range.filter(function(q: Nat) { (q * m + r).coprime(m * n) }).length =
            n.totient
        forall(q: Nat) {
            if n.range.contains(q) {
                totient_row_pred(m, n, r)(q) = (q * m + r).coprime(m * n)
            }
        }
        filter_length_of_pointwise[Nat](n.range, totient_row_pred(m, n, r),
            function(q: Nat) { (q * m + r).coprime(m * n) })
        n.range.filter(totient_row_pred(m, n, r)).length =
            n.range.filter(function(q: Nat) { (q * m + r).coprime(m * n) }).length
        n.range.filter(totient_row_pred(m, n, r)).length = n.totient
    }
}

/// A non-coprime row contributes no selected entries.
lemma totient_row_pred_filter_length_if_not_coprime(m: Nat, n: Nat, r: Nat) {
    not r.coprime(m) implies n.range.filter(totient_row_pred(m, n, r)).length = Nat.0
} by {
    if not r.coprime(m) {
        row_coprime_product_filter_length_if_not_coprime(m, n, r)
        n.range.filter(function(q: Nat) { (q * m + r).coprime(m * n) }).length =
            Nat.0
        forall(q: Nat) {
            if n.range.contains(q) {
                totient_row_pred(m, n, r)(q) = (q * m + r).coprime(m * n)
            }
        }
        filter_length_of_pointwise[Nat](n.range, totient_row_pred(m, n, r),
            function(q: Nat) { (q * m + r).coprime(m * n) })
        n.range.filter(totient_row_pred(m, n, r)).length =
            n.range.filter(function(q: Nat) { (q * m + r).coprime(m * n) }).length
        n.range.filter(totient_row_pred(m, n, r)).length = Nat.0
    }
}

/// The row-filter length in the product rectangle is the corresponding
/// totient row contribution.
lemma totient_product_row_filter_length(m: Nat, n: Nat, r: Nat) {
    n != Nat.0 and m.coprime(n) implies
        list_pair_product_row_filter_length_fn[Nat, Nat](
            n.range,
            totient_product_pair_pred(m, n)
        )(r) = totient_row_contribution(m, n)(r)
} by {
    if n != Nat.0 and m.coprime(n) {
        list_pair_product_row_filter_length_fn[Nat, Nat](
            n.range, totient_product_pair_pred(m, n))(r) =
            list_pair_with_left(r, n.range).filter(totient_product_pair_pred(m, n)).length
        list_pair_with_left_filter_length[Nat, Nat](r, n.range, totient_product_pair_pred(m, n))
        list_pair_with_left(r, n.range).filter(totient_product_pair_pred(m, n)).length =
            n.range.filter(function(q: Nat) {
                totient_product_pair_pred(m, n)(Pair.new(r, q))
            }).length
        totient_product_pair_row_pred(m, n, r)
        forall(q: Nat) {
            if n.range.contains(q) {
                function(x: Nat) { totient_product_pair_pred(m, n)(Pair.new(r, x)) }(q) =
                    totient_row_pred(m, n, r)(q)
            }
        }
        filter_length_of_pointwise[Nat](n.range, function(q: Nat) {
                totient_product_pair_pred(m, n)(Pair.new(r, q))
            },
            totient_row_pred(m, n, r))
        n.range.filter(function(q: Nat) {
                totient_product_pair_pred(m, n)(Pair.new(r, q))
            }).length =
            n.range.filter(totient_row_pred(m, n, r)).length
        if r.coprime(m) {
            totient_row_pred_filter_length_if_coprime(m, n, r)
            n.range.filter(totient_row_pred(m, n, r)).length = n.totient
            list_pair_product_row_filter_length_fn[Nat, Nat](
                n.range, totient_product_pair_pred(m, n))(r) =
                n.range.filter(totient_row_pred(m, n, r)).length
            list_pair_product_row_filter_length_fn[Nat, Nat](
                n.range, totient_product_pair_pred(m, n))(r) = n.totient
            totient_row_contribution(m, n)(r) = n.totient
            list_pair_product_row_filter_length_fn[Nat, Nat](
                n.range, totient_product_pair_pred(m, n))(r) =
                totient_row_contribution(m, n)(r)
        }
        if not r.coprime(m) {
            totient_row_pred_filter_length_if_not_coprime(m, n, r)
            n.range.filter(totient_row_pred(m, n, r)).length = Nat.0
            list_pair_product_row_filter_length_fn[Nat, Nat](
                n.range, totient_product_pair_pred(m, n))(r) =
                n.range.filter(totient_row_pred(m, n, r)).length
            list_pair_product_row_filter_length_fn[Nat, Nat](
                n.range, totient_product_pair_pred(m, n))(r) = Nat.0
            totient_row_contribution(m, n)(r) = Nat.0
            list_pair_product_row_filter_length_fn[Nat, Nat](
                n.range, totient_product_pair_pred(m, n))(r) =
                totient_row_contribution(m, n)(r)
        }
        list_pair_product_row_filter_length_fn[Nat, Nat](
            n.range, totient_product_pair_pred(m, n))(r) =
            totient_row_contribution(m, n)(r)
    }
}

/// Euler's totient is multiplicative on positive coprime arguments.
theorem totient_mul_coprime_positive(m: Nat, n: Nat) {
    Nat.0 < m and Nat.0 < n and m.coprime(n) implies
        (m * n).totient = m.totient * n.totient
} by {
    if Nat.0 < m and Nat.0 < n and m.coprime(n) {
        if m = Nat.0 {
            Nat.0 < Nat.0
            not_lt_zero(Nat.0)
            false
        }
        if n = Nat.0 {
            Nat.0 < Nat.0
            not_lt_zero(Nat.0)
            false
        }
        m != Nat.0
        n != Nat.0

        row_encode_filter_length_range(m, n, totient_product_pred(m, n))
        list_pair_product[Nat, Nat](m.range, n.range).filter(totient_product_pair_pred(m, n)).length =
            (m * n).range.filter(totient_product_pred(m, n)).length

        list_pair_product_filter_length_sum_rows[Nat, Nat](
            m.range, n.range, totient_product_pair_pred(m, n))
        list_pair_product[Nat, Nat](m.range, n.range).filter(totient_product_pair_pred(m, n)).length =
            sum[Nat](map[Nat, Nat](m.range,
                list_pair_product_row_filter_length_fn[Nat, Nat](
                    n.range, totient_product_pair_pred(m, n))))

        let row_fn: Nat -> Nat =
            list_pair_product_row_filter_length_fn[Nat, Nat](
                n.range, totient_product_pair_pred(m, n))
        let contribution: Nat -> Nat = totient_row_contribution(m, n)
        forall(r: Nat) {
            totient_product_row_filter_length(m, n, r)
            row_fn(r) = contribution(r)
        }
        function_extensionality(row_fn, contribution)
        row_fn = contribution
        map[Nat, Nat](m.range, row_fn) = map[Nat, Nat](m.range, contribution)
        sum[Nat](map[Nat, Nat](m.range, row_fn)) =
            sum[Nat](map[Nat, Nat](m.range, contribution))

        sum_filter_indicator_value[Nat](m.range, n.totient, coprime_to_pred(m))
        sum[Nat](map[Nat, Nat](m.range, contribution)) =
            n.totient * m.range.filter(coprime_to_pred(m)).length

        range_filter_coprime_length(m)
        m.range.filter(function(x: Nat) { x.coprime(m) }).length = m.totient
        forall(x: Nat) {
            if m.range.contains(x) {
                coprime_to_pred(m)(x) = x.coprime(m)
            }
        }
        filter_length_of_pointwise[Nat](m.range, coprime_to_pred(m),
            function(x: Nat) { x.coprime(m) })
        m.range.filter(coprime_to_pred(m)).length =
            m.range.filter(function(x: Nat) { x.coprime(m) }).length
        m.range.filter(coprime_to_pred(m)).length = m.totient
        sum[Nat](map[Nat, Nat](m.range, contribution)) = n.totient * m.totient

        list_pair_product[Nat, Nat](m.range, n.range).filter(totient_product_pair_pred(m, n)).length =
            n.totient * m.totient
        (m * n).range.filter(totient_product_pred(m, n)).length = n.totient * m.totient

        range_filter_coprime_length(m * n)
        (m * n).range.filter(function(x: Nat) { x.coprime(m * n) }).length = (m * n).totient
        forall(x: Nat) {
            if (m * n).range.contains(x) {
                totient_product_pred(m, n)(x) = x.coprime(m * n)
            }
        }
        filter_length_of_pointwise[Nat]((m * n).range, totient_product_pred(m, n),
            function(x: Nat) { x.coprime(m * n) })
        (m * n).range.filter(totient_product_pred(m, n)).length =
            (m * n).range.filter(function(x: Nat) { x.coprime(m * n) }).length
        (m * n).range.filter(totient_product_pred(m, n)).length = (m * n).totient
        (m * n).totient = n.totient * m.totient
        n.totient * m.totient = m.totient * n.totient
        (m * n).totient = m.totient * n.totient
    }
}

/// Euler's totient is multiplicative on one pair of coprime arguments.
theorem nat_totient_mul_coprime(a: Nat, b: Nat) {
    a.coprime(b) implies nat_totient(a * b) = nat_totient(a) * nat_totient(b)
} by {
    if a.coprime(b) {
        if a = Nat.0 {
            coprime_zero_left_imp_one(b)
            b = Nat.1
            a * b = Nat.0
            nat_totient_zero
            nat_totient(a * b) = Nat.0
            nat_totient(a) = Nat.0
            nat_totient(b) = Nat.1
            nat_totient(a) * nat_totient(b) = Nat.0
            nat_totient(a * b) = nat_totient(a) * nat_totient(b)
        } else {
            if b = Nat.0 {
                coprime_zero_right_imp_one(a)
                a = Nat.1
                a * b = Nat.0
                nat_totient_zero
                nat_totient(a * b) = Nat.0
                nat_totient(a) = Nat.1
                nat_totient(b) = Nat.0
                nat_totient(a) * nat_totient(b) = Nat.0
                nat_totient(a * b) = nat_totient(a) * nat_totient(b)
            } else {
                Nat.0 < a
                Nat.0 < b
                totient_mul_coprime_positive(a, b)
                (a * b).totient = a.totient * b.totient
                nat_totient(a * b) = (a * b).totient
                nat_totient(a) = a.totient
                nat_totient(b) = b.totient
                nat_totient(a * b) = nat_totient(a) * nat_totient(b)
            }
        }
    }
}
