from combinatorics import binom, binomial_term, binomial, choose_zero, choose_n
from nat import Nat
from number_theory.factorisation import prime_does_not_divide_one
from nat import gcd_of_prime
from number_theory.coprime import coprime_divides_of_divides_mul
from nat import factorial_step, divides_lte, divides_add, divides_mul, div_imp_mod,
    suc_sub_one, add_imp_sub, lte_ref, not_lt_zero, add_one_right, add_zero_left,
    mul_one_left
from nat import exp_zero, exp_one, exp_add, zero_exp, one_exp
from number_theory.congruence import congr_mod_add, congr_mod_mul, congr_mod_refl, congr_mod_symm, congr_mod_trans
from number_theory.modular_inverse import cancel_coprime
from list import partial, partial_split_first_last, partial_split_last, partial_zero
from data.basic.functions import compose
numerals Nat

/// A prime never divides any natural number strictly smaller than itself.
theorem prime_does_not_divide_below(p: Nat, m: Nat) {
    p.is_prime and Nat.0 < m and m < p implies not p.divides(m)
} by {
    if p.is_prime and Nat.0 < m and m < p {
        if p.divides(m) {
            divides_lte(p, m)
            m = Nat.0 or p <= m
            not (m = Nat.0)
            p <= m
            not (m < p)
            false
        }
    }
}

/// Step lemma for prime_does_not_divide_factorial: if p does not divide k!
/// and p does not divide k.suc, then p does not divide k.suc.factorial.
theorem prime_not_divides_factorial_step(p: Nat, k: Nat) {
    p.is_prime and not p.divides(k.factorial) and not p.divides(k.suc)
        implies not p.divides(k.suc.factorial)
} by {
    if p.is_prime and not p.divides(k.factorial) and not p.divides(k.suc) {
        factorial_step(k)
        k.suc.factorial = k.suc * k.factorial
        if p.divides(k.suc.factorial) {
            p.divides(k.suc * k.factorial)
            // Prime Euclid's lemma: p | k.suc or p | k!.
            gcd_of_prime(p, k.suc)
            if p.divides(k.suc) {
                false
            } else {
                p.gcd(k.suc) = Nat.1
                p.coprime(k.suc)
                coprime_divides_of_divides_mul(p, k.suc, k.factorial)
                p.divides(k.factorial)
                false
            }
        }
    }
}

/// Inductive predicate used in prime_does_not_divide_factorial: at each Nat x,
/// p does not divide x.factorial whenever x < p.
define p_no_div_fact_pred(p: Nat) -> (Nat -> Bool) {
    function(x: Nat) {
        x < p implies not p.divides(x.factorial)
    }
}

/// Base case for prime_does_not_divide_factorial: p does not divide 0! = 1.
theorem prime_no_div_fact_zero(p: Nat) {
    p.is_prime implies p_no_div_fact_pred(p)(Nat.0)
} by {
    if p.is_prime {
        Nat.0.factorial = Nat.1
        prime_does_not_divide_one(p)
        if Nat.0 < p {
            not p.divides(Nat.0.factorial)
        }
    }
}

/// Inductive step for prime_does_not_divide_factorial.
theorem prime_no_div_fact_step(p: Nat, k: Nat) {
    p.is_prime and p_no_div_fact_pred(p)(k) implies p_no_div_fact_pred(p)(k.suc)
} by {
    if p.is_prime and p_no_div_fact_pred(p)(k) {
        if k.suc < p {
            k < k.suc
            k < p
            not p.divides(k.factorial)
            Nat.0 < k.suc
            prime_does_not_divide_below(p, k.suc)
            not p.divides(k.suc)
            prime_not_divides_factorial_step(p, k)
            not p.divides(k.suc.factorial)
        }
    }
}

/// A prime never divides the factorial of any natural number strictly smaller
/// than itself.
theorem prime_does_not_divide_factorial(p: Nat, m: Nat) {
    p.is_prime and m < p implies not p.divides(m.factorial)
} by {
    if p.is_prime and m < p {
        let f: Nat -> Bool = p_no_div_fact_pred(p)
        prime_no_div_fact_zero(p)
        f(Nat.0)
        forall(k: Nat) {
            if f(k) {
                prime_no_div_fact_step(p, k)
                f(k.suc)
            }
        }
        f(m)
        m < p implies not p.divides(m.factorial)
        not p.divides(m.factorial)
    }
}

/// Prime Euclid's lemma: if a prime divides a product, it divides at least one
/// factor.
theorem prime_divides_mul(p: Nat, a: Nat, b: Nat) {
    p.is_prime and p.divides(a * b) implies p.divides(a) or p.divides(b)
} by {
    if p.is_prime and p.divides(a * b) {
        gcd_of_prime(p, a)
        if p.gcd(a) = Nat.1 {
            p.coprime(a)
            coprime_divides_of_divides_mul(p, a, b)
            p.divides(b)
        }
    }
}

/// A prime divides p! (its own factorial).
theorem prime_divides_self_factorial(p: Nat) {
    p.is_prime implies p.divides(p.factorial)
} by {
    if p.is_prime {
        Nat.1 < p
        Nat.0 < p
        let pred: Nat satisfy { pred.suc = p }
        factorial_step(pred)
        pred.suc.factorial = pred.suc * pred.factorial
        p.factorial = p * pred.factorial
        p * pred.factorial = pred.factorial * p
        p.factorial = pred.factorial * p
        p.divides(p.factorial)
    }
}

/// For prime p and 0 < k < p, the binomial coefficient binom(p, k) is divisible
/// by p. Cornerstone lemma for the Freshman's dream and Fermat's little theorem.
theorem prime_divides_binom(p: Nat, k: Nat) {
    p.is_prime and Nat.0 < k and k < p implies p.divides(p.binom(k))
} by {
    if p.is_prime and Nat.0 < k and k < p {
        // Defining identity for binom(p, k) when k <= p.
        k <= p
        not (p < k)
        p.binom(k) * k.factorial * (p - k).factorial = p.factorial
        // p divides p!
        prime_divides_self_factorial(p)
        p.divides(p.factorial)
        p.divides(p.binom(k) * k.factorial * (p - k).factorial)
        // Apply prime Euclid's lemma to ((p.binom(k) * k!) * (p-k)!).
        prime_divides_mul(p, p.binom(k) * k.factorial, (p - k).factorial)
        if p.divides((p - k).factorial) {
            // 0 < k implies p - k < p
            Nat.0 < k
            let kp: Nat satisfy { kp.suc = k }
            // p = (p - k) + k, so (p - k) + 1 <= p
            (p - k) + k = p
            (p - k) + kp.suc = p
            ((p - k) + kp).suc = p
            (p - k) < ((p - k) + kp).suc
            (p - k) < p
            prime_does_not_divide_factorial(p, p - k)
            not p.divides((p - k).factorial)
            false
        }
        p.divides(p.binom(k) * k.factorial)
        prime_divides_mul(p, p.binom(k), k.factorial)
        if p.divides(k.factorial) {
            prime_does_not_divide_factorial(p, k)
            not p.divides(k.factorial)
            false
        }
        p.divides(p.binom(k))
    }
}

/// Divisibility predicate for `divides_partial_nat`: at each Nat m, if d
/// divides every f(k) for k < m, then d divides partial(f, m).
define divides_partial_pred(d: Nat, f: Nat -> Nat) -> (Nat -> Bool) {
    function(m: Nat) {
        (forall(k: Nat) { k < m implies d.divides(f(k)) })
            implies d.divides(partial[Nat](f, m))
    }
}

/// Base case for divides_partial_nat: partial(f, 0) is the zero element,
/// which every d divides.
theorem divides_partial_nat_zero(d: Nat, f: Nat -> Nat) {
    divides_partial_pred(d, f)(Nat.0)
} by {
    partial[Nat](f, Nat.0) = Nat.0
    d.divides(Nat.0)
}

/// Inductive step for divides_partial_nat.
theorem divides_partial_nat_step(d: Nat, f: Nat -> Nat, m: Nat) {
    divides_partial_pred(d, f)(m)
        implies divides_partial_pred(d, f)(m.suc)
} by {
    if divides_partial_pred(d, f)(m) {
        if (forall(k: Nat) { k < m.suc implies d.divides(f(k)) }) {
            forall(k: Nat) {
                if k < m {
                    k < m.suc
                    d.divides(f(k))
                }
            }
            d.divides(partial[Nat](f, m))
            m < m.suc
            d.divides(f(m))
            partial_split_last[Nat](f, m)
            partial[Nat](f, m.suc) = partial[Nat](f, m) + f(m)
            divides_add(partial[Nat](f, m), f(m), d)
            d.divides(partial[Nat](f, m.suc))
        }
    }
}

/// Helper: p - 1 = pp when pp.suc = p.
theorem sub_one_pred(p: Nat, pp: Nat) {
    pp.suc = p implies p - Nat.1 = pp
} by {
    if pp.suc = p {
        pp.suc = pp + Nat.1
        pp + Nat.1 = p
        add_imp_sub(pp, Nat.1, p)
    }
}

/// Helper: p.suc - 2 = pp when pp.suc = p.
theorem suc_sub_two_pred(p: Nat, pp: Nat) {
    pp.suc = p implies p.suc - Nat.2 = pp
} by {
    if pp.suc = p {
        p.suc = pp.suc.suc
        pp.suc.suc = pp + Nat.2
        pp + Nat.2 = p.suc
        add_imp_sub(pp, Nat.2, p.suc)
    }
}

/// Helper: p.suc - 2 = p - 1 when 1 < p.
theorem suc_sub_two_eq_sub_one(p: Nat) {
    Nat.1 < p implies p.suc - Nat.2 = p - Nat.1
} by {
    if Nat.1 < p {
        let pp: Nat satisfy { pp.suc = p }
        sub_one_pred(p, pp)
        p - Nat.1 = pp
        suc_sub_two_pred(p, pp)
        p.suc - Nat.2 = pp
        p.suc - Nat.2 = p - Nat.1
    }
}

/// Helper: p.suc - 1 = p.
theorem suc_sub_one_eq(p: Nat) {
    p.suc - Nat.1 = p
} by {
    suc_sub_one(p)
}

/// Helper: p.suc >= 2 when 1 < p.
theorem suc_geq_two(p: Nat) {
    Nat.1 < p implies p.suc >= Nat.2
} by {
    if Nat.1 < p {
        Nat.2 <= p
        p <= p.suc
        Nat.2 <= p.suc
    }
}

/// A Nat function whose first n values are all divisible by d sums to a value
/// divisible by d.
theorem divides_partial_nat(d: Nat, f: Nat -> Nat, n: Nat) {
    (forall(k: Nat) { k < n implies d.divides(f(k)) })
        implies d.divides(partial[Nat](f, n))
} by {
    let q: Nat -> Bool = divides_partial_pred(d, f)
    divides_partial_nat_zero(d, f)
    q(Nat.0)
    forall(m: Nat) {
        if q(m) {
            divides_partial_nat_step(d, f, m)
            q(m.suc)
        }
    }
    q(n)
}

/// For prime p and 0 < k < p, p divides the k-th binomial term in the expansion
/// of (a + b).pow(p).
theorem prime_divides_binomial_term_middle(p: Nat, a: Nat, b: Nat, k: Nat) {
    p.is_prime and Nat.0 < k and k < p
        implies p.divides(binomial_term(a, b, p, k))
} by {
    if p.is_prime and Nat.0 < k and k < p {
        prime_divides_binom(p, k)
        p.divides(p.binom(k))
        // binomial_term(a, b, p, k) = p.binom(k) * a.pow(k) * b.pow(p - k)
        binomial_term(a, b, p, k) = p.binom(k) * a.pow(k) * b.pow(p - k)
        divides_mul(p.binom(k), a.pow(k), p)
        p.divides(p.binom(k) * a.pow(k))
        divides_mul(p.binom(k) * a.pow(k), b.pow(p - k), p)
        p.divides(p.binom(k) * a.pow(k) * b.pow(p - k))
    }
}

/// The k=0 term of (a + b).pow(p) expansion equals b.pow(p).
theorem binomial_term_left_boundary(a: Nat, b: Nat, p: Nat) {
    binomial_term(a, b, p, Nat.0) = b.pow(p)
} by {
    choose_zero(p)
    p.binom(Nat.0) = Nat.1
    exp_zero(a)
    a.pow(Nat.0) = Nat.1
    p - Nat.0 = p
    binomial_term(a, b, p, Nat.0) = Nat.1 * Nat.1 * b.pow(p)
}

/// The k=p term of (a + b).pow(p) expansion equals a.pow(p).
theorem binomial_term_right_boundary(a: Nat, b: Nat, p: Nat) {
    binomial_term(a, b, p, p) = a.pow(p)
} by {
    choose_n(p)
    p.binom(p) = Nat.1
    p - p = Nat.0
    exp_zero(b)
    b.pow(Nat.0) = Nat.1
    binomial_term(a, b, p, p) = Nat.1 * a.pow(p) * Nat.1
}

/// Helper: each shifted-index binomial term in the middle is divisible by p.
theorem shifted_binom_term_div(p: Nat, a: Nat, b: Nat, k: Nat) {
    p.is_prime and k < p - Nat.1
        implies p.divides(compose(binomial_term(a, b, p), Nat.suc)(k))
} by {
    if p.is_prime and k < p - Nat.1 {
        Nat.1 < p
        let pp: Nat satisfy { pp.suc = p }
        sub_one_pred(p, pp)
        p - Nat.1 = pp
        k < pp
        Nat.0 < k.suc
        k.suc <= pp
        k.suc < pp.suc
        k.suc < p
        prime_divides_binomial_term_middle(p, a, b, k.suc)
        p.divides(binomial_term(a, b, p, k.suc))
        compose(binomial_term(a, b, p), Nat.suc)(k) = binomial_term(a, b, p, k.suc)
        p.divides(compose(binomial_term(a, b, p), Nat.suc)(k))
    }
}

/// The middle of the binomial expansion of (a + b).pow(p) when p is prime is
/// divisible by p.
theorem prime_divides_binomial_middle(p: Nat, a: Nat, b: Nat) {
    p.is_prime implies
        p.divides(partial[Nat](compose(binomial_term(a, b, p), Nat.suc), p - Nat.1))
} by {
    if p.is_prime {
        forall(k: Nat) {
            if k < p - Nat.1 {
                shifted_binom_term_div(p, a, b, k)
                p.divides(compose(binomial_term(a, b, p), Nat.suc)(k))
            }
        }
        divides_partial_nat(p, compose(binomial_term(a, b, p), Nat.suc), p - Nat.1)
    }
}


/// Splitting (a + b).pow(p) using the binomial theorem with first/last terms
/// peeled off: the expansion equals b.pow(p) + middle + a.pow(p).
theorem binomial_pow_split(a: Nat, b: Nat, p: Nat) {
    Nat.1 < p implies
        (a + b).pow(p) =
            b.pow(p) + partial[Nat](compose(binomial_term(a, b, p), Nat.suc), p - Nat.1) + a.pow(p)
} by {
    if Nat.1 < p {
        suc_sub_two_eq_sub_one(p)
        p.suc - Nat.2 = p - Nat.1
        suc_sub_one_eq(p)
        p.suc - Nat.1 = p
        suc_geq_two(p)
        p.suc >= Nat.2
        binomial(a, b, p)
        (a + b).pow(p) = partial[Nat](binomial_term(a, b, p), p.suc)
        partial_split_first_last[Nat](binomial_term(a, b, p), p.suc)
        partial[Nat](binomial_term(a, b, p), p.suc) =
            binomial_term(a, b, p, Nat.0) + partial[Nat](compose(binomial_term(a, b, p), Nat.suc), p.suc - Nat.2) + binomial_term(a, b, p, p.suc - Nat.1)
        partial[Nat](binomial_term(a, b, p), p.suc) =
            binomial_term(a, b, p, Nat.0) + partial[Nat](compose(binomial_term(a, b, p), Nat.suc), p - Nat.1) + binomial_term(a, b, p, p)
        binomial_term_left_boundary(a, b, p)
        binomial_term(a, b, p, Nat.0) = b.pow(p)
        binomial_term_right_boundary(a, b, p)
        binomial_term(a, b, p, p) = a.pow(p)
        partial[Nat](binomial_term(a, b, p), p.suc) =
            b.pow(p) + partial[Nat](compose(binomial_term(a, b, p), Nat.suc), p - Nat.1) + a.pow(p)
    }
}

/// If d divides x, then x is congruent to 0 modulo d.
theorem divides_imp_congr_zero(d: Nat, x: Nat) {
    d.divides(x) implies x.congr_mod(Nat.0, d)
} by {
    if d.divides(x) {
        div_imp_mod(x, d)
        x.mod(d) = Nat.0
        Nat.0.mod(d) = Nat.0
        x.mod(d) = Nat.0.mod(d)
    }
}

/// Freshman's dream: for any prime p,
///   (a + b).pow(p) is congruent to a.pow(p) + b.pow(p) modulo p.
theorem freshmans_dream(p: Nat, a: Nat, b: Nat) {
    p.is_prime implies (a + b).pow(p).congr_mod(a.pow(p) + b.pow(p), p)
} by {
    if p.is_prime {
        Nat.1 < p
        binomial_pow_split(a, b, p)
        let middle: Nat = partial[Nat](compose(binomial_term(a, b, p), Nat.suc), p - Nat.1)
        (a + b).pow(p) = b.pow(p) + middle + a.pow(p)
        prime_divides_binomial_middle(p, a, b)
        p.divides(middle)
        divides_imp_congr_zero(p, middle)
        middle.congr_mod(Nat.0, p)
        // (a+b)^p = b^p + middle + a^p ≡ b^p + 0 + a^p = a^p + b^p (mod p)
        congr_mod_refl(b.pow(p) + a.pow(p), p)
        congr_mod_add(b.pow(p) + a.pow(p), middle, b.pow(p) + a.pow(p), Nat.0, p)
        // wrong direction: I want b^p + middle + a^p ≡ b^p + 0 + a^p
        // Use (b^p + a^p) + middle ≡ (b^p + a^p) + 0 = b^p + a^p
        ((b.pow(p) + a.pow(p)) + middle).congr_mod(b.pow(p) + a.pow(p) + Nat.0, p)
        b.pow(p) + a.pow(p) + Nat.0 = b.pow(p) + a.pow(p)
        ((b.pow(p) + a.pow(p)) + middle).congr_mod(b.pow(p) + a.pow(p), p)
        // Rewrite (b^p + a^p) + middle = b^p + middle + a^p via assoc + comm.
        b.pow(p) + a.pow(p) + middle = b.pow(p) + (a.pow(p) + middle)
        a.pow(p) + middle = middle + a.pow(p)
        b.pow(p) + (a.pow(p) + middle) = b.pow(p) + (middle + a.pow(p))
        b.pow(p) + (middle + a.pow(p)) = b.pow(p) + middle + a.pow(p)
        b.pow(p) + a.pow(p) + middle = b.pow(p) + middle + a.pow(p)
        (b.pow(p) + middle + a.pow(p)).congr_mod(b.pow(p) + a.pow(p), p)
        ((a + b).pow(p)).congr_mod(b.pow(p) + a.pow(p), p)
        b.pow(p) + a.pow(p) = a.pow(p) + b.pow(p)
        ((a + b).pow(p)).congr_mod(a.pow(p) + b.pow(p), p)
    }
}

/// Inductive predicate for Fermat's little theorem: a.pow(p) is congruent to a
/// modulo p.
define fermat_pred(p: Nat) -> (Nat -> Bool) {
    function(a: Nat) {
        a.pow(p).congr_mod(a, p)
    }
}

/// Base case for Fermat's little theorem: 0.pow(p) ≡ 0 (mod p).
theorem fermat_base(p: Nat) {
    p.is_prime implies fermat_pred(p)(Nat.0)
} by {
    if p.is_prime {
        Nat.1 < p
        p != Nat.0
        zero_exp(p)
        Nat.0.pow(p) = Nat.0
        congr_mod_refl(Nat.0, p)
        Nat.0.congr_mod(Nat.0, p)
        Nat.0.pow(p).congr_mod(Nat.0, p)
    }
}

/// Inductive step for Fermat's little theorem: if a.pow(p) ≡ a (mod p), then
/// (a + 1).pow(p) ≡ a + 1 (mod p).
theorem fermat_step(p: Nat, a: Nat) {
    p.is_prime and fermat_pred(p)(a) implies fermat_pred(p)(a.suc)
} by {
    if p.is_prime and fermat_pred(p)(a) {
        a.pow(p).congr_mod(a, p)
        // Freshman: (a + 1).pow(p) ≡ a.pow(p) + 1.pow(p) (mod p).
        freshmans_dream(p, a, Nat.1)
        ((a + Nat.1).pow(p)).congr_mod(a.pow(p) + Nat.1.pow(p), p)
        one_exp(p)
        Nat.1.pow(p) = Nat.1
        a.pow(p) + Nat.1.pow(p) = a.pow(p) + Nat.1
        ((a + Nat.1).pow(p)).congr_mod(a.pow(p) + Nat.1, p)
        // Add 1 ≡ 1 to a.pow(p) ≡ a to get a.pow(p) + 1 ≡ a + 1.
        congr_mod_refl(Nat.1, p)
        Nat.1.congr_mod(Nat.1, p)
        congr_mod_add(a.pow(p), Nat.1, a, Nat.1, p)
        (a.pow(p) + Nat.1).congr_mod(a + Nat.1, p)
        congr_mod_trans((a + Nat.1).pow(p), a.pow(p) + Nat.1, a + Nat.1, p)
        ((a + Nat.1).pow(p)).congr_mod(a + Nat.1, p)
        a + Nat.1 = a.suc
        (a + Nat.1).pow(p) = a.suc.pow(p)
        a.suc.pow(p).congr_mod(a.suc, p)
    }
}

/// Bridge: fermat_pred unfolds to the underlying congruence.
theorem fermat_pred_unfold(p: Nat, a: Nat) {
    fermat_pred(p)(a) = a.pow(p).congr_mod(a, p)
}

/// Inner induction step for Fermat: completes the inductive run on the
/// fermat_pred predicate.
theorem fermats_little_run(p: Nat, a: Nat) {
    p.is_prime implies fermat_pred(p)(a)
} by {
    if p.is_prime {
        let f: Nat -> Bool = function(x: Nat) {
            x.pow(p).congr_mod(x, p)
        }
        forall(y: Nat) {
            f(y) = fermat_pred(p)(y)
            fermat_pred(p)(y) = f(y)
        }
        // Base
        fermat_base(p)
        fermat_pred(p)(Nat.0)
        f(Nat.0)
        // Step
        forall(x: Nat) {
            if f(x) {
                fermat_pred(p)(x)
                fermat_step(p, x)
                fermat_pred(p)(x.suc)
                f(x.suc)
            }
        }
        // Conclusion
        f(a)
        fermat_pred(p)(a)
    }
}

/// Fermat's little theorem in congruence form: for prime p and any natural a,
///   a.pow(p) ≡ a  (mod p).
theorem fermats_little_congr(p: Nat, a: Nat) {
    p.is_prime implies a.pow(p).congr_mod(a, p)
} by {
    fermat_pred_unfold(p, a)
    if p.is_prime {
        fermats_little_run(p, a)
        fermat_pred(p)(a)
        a.pow(p).congr_mod(a, p)
    }
}

/// Fermat's little theorem: for prime p and any natural a, the remainder of
/// a.pow(p) modulo p equals the remainder of a modulo p.
theorem fermats_little(p: Nat, a: Nat) {
    p.is_prime implies a.pow(p).mod(p) = a.mod(p)
} by {
    if p.is_prime {
        fermats_little_congr(p, a)
    }
}

/// Helper: m * m.pow(p - 1) = m.pow(p) for prime p, using p = (p - 1) + 1.
theorem mul_pow_pred_eq_pow(m: Nat, p: Nat) {
    p.is_prime implies m * m.pow(p - Nat.1) = m.pow(p)
} by {
    if p.is_prime {
        Nat.1 < p
        let pp: Nat satisfy { pp.suc = p }
        sub_one_pred(p, pp)
        p - Nat.1 = pp
        // m.pow(p) = m.pow(pp.suc) = m.pow(pp + 1) = m.pow(pp) * m.pow(1) = m.pow(pp) * m.
        pp + Nat.1 = pp.suc
        pp + Nat.1 = p
        exp_add(m, pp, Nat.1)
        m.pow(pp + Nat.1) = m.pow(pp) * m.pow(Nat.1)
        m.pow(p) = m.pow(pp) * m.pow(Nat.1)
        exp_one(m)
        m.pow(Nat.1) = m
        m.pow(p) = m.pow(pp) * m
        m * m.pow(pp) = m.pow(pp) * m
        m * m.pow(p - Nat.1) = m.pow(p)
    }
}

/// Inductive predicate for rsa_pow_congr.
define rsa_pow_pred(m: Nat, p: Nat) -> (Nat -> Bool) {
    function(k: Nat) {
        m.pow(k * (p - Nat.1) + Nat.1).congr_mod(m, p)
    }
}

/// Base case for rsa_pow_congr: at k = 0, m.pow(1) = m.
theorem rsa_pow_base(m: Nat, p: Nat) {
    rsa_pow_pred(m, p)(Nat.0)
} by {
    Nat.0 * (p - Nat.1) = Nat.0
    Nat.0 + Nat.1 = Nat.1
    Nat.0 * (p - Nat.1) + Nat.1 = Nat.1
    exp_one(m)
    m.pow(Nat.1) = m
    m.pow(Nat.0 * (p - Nat.1) + Nat.1) = m
    congr_mod_refl(m, p)
}

/// Inductive step for rsa_pow_congr: m.pow((k+1)(p-1)+1) ≡ m (mod p) given the
/// inductive hypothesis at k.
theorem rsa_pow_step(m: Nat, p: Nat, k: Nat) {
    p.is_prime and rsa_pow_pred(m, p)(k) implies rsa_pow_pred(m, p)(k.suc)
} by {
    if p.is_prime and rsa_pow_pred(m, p)(k) {
        m.pow(k * (p - Nat.1) + Nat.1).congr_mod(m, p)
        // Express the new exponent.
        k.suc = k + Nat.1
        k.suc * (p - Nat.1) = (k + Nat.1) * (p - Nat.1)
        (k + Nat.1) * (p - Nat.1) = k * (p - Nat.1) + Nat.1 * (p - Nat.1)
        Nat.1 * (p - Nat.1) = p - Nat.1
        k.suc * (p - Nat.1) = k * (p - Nat.1) + (p - Nat.1)
        // Add the trailing +1 and reorder.
        k.suc * (p - Nat.1) + Nat.1 = k * (p - Nat.1) + (p - Nat.1) + Nat.1
        k * (p - Nat.1) + (p - Nat.1) + Nat.1 = k * (p - Nat.1) + ((p - Nat.1) + Nat.1)
        (p - Nat.1) + Nat.1 = Nat.1 + (p - Nat.1)
        k * (p - Nat.1) + ((p - Nat.1) + Nat.1) = k * (p - Nat.1) + (Nat.1 + (p - Nat.1))
        k * (p - Nat.1) + (Nat.1 + (p - Nat.1)) = k * (p - Nat.1) + Nat.1 + (p - Nat.1)
        k.suc * (p - Nat.1) + Nat.1 = k * (p - Nat.1) + Nat.1 + (p - Nat.1)
        k.suc * (p - Nat.1) + Nat.1 = (k * (p - Nat.1) + Nat.1) + (p - Nat.1)
        // Split the power.
        exp_add(m, k * (p - Nat.1) + Nat.1, p - Nat.1)
        m.pow((k * (p - Nat.1) + Nat.1) + (p - Nat.1)) =
            m.pow(k * (p - Nat.1) + Nat.1) * m.pow(p - Nat.1)
        m.pow(k.suc * (p - Nat.1) + Nat.1) =
            m.pow(k * (p - Nat.1) + Nat.1) * m.pow(p - Nat.1)
        // Multiply the IH congruence by m.pow(p - 1) ≡ m.pow(p - 1).
        congr_mod_refl(m.pow(p - Nat.1), p)
        congr_mod_mul(
            m.pow(k * (p - Nat.1) + Nat.1), m.pow(p - Nat.1),
            m, m.pow(p - Nat.1), p)
        (m.pow(k * (p - Nat.1) + Nat.1) * m.pow(p - Nat.1)).congr_mod(
            m * m.pow(p - Nat.1), p)
        m.pow(k.suc * (p - Nat.1) + Nat.1).congr_mod(m * m.pow(p - Nat.1), p)
        // Simplify m * m.pow(p - 1) to m.pow(p) and apply Fermat.
        mul_pow_pred_eq_pow(m, p)
        m * m.pow(p - Nat.1) = m.pow(p)
        m.pow(k.suc * (p - Nat.1) + Nat.1).congr_mod(m.pow(p), p)
        fermats_little_congr(p, m)
        m.pow(p).congr_mod(m, p)
        congr_mod_trans(m.pow(k.suc * (p - Nat.1) + Nat.1), m.pow(p), m, p)
        m.pow(k.suc * (p - Nat.1) + Nat.1).congr_mod(m, p)
    }
}

/// Inner induction for rsa_pow_congr: runs the induction over k.
theorem rsa_pow_run(p: Nat, m: Nat, k: Nat) {
    p.is_prime implies rsa_pow_pred(m, p)(k)
} by {
    if p.is_prime {
        let f: Nat -> Bool = function(x: Nat) {
            m.pow(x * (p - Nat.1) + Nat.1).congr_mod(m, p)
        }
        forall(y: Nat) {
            rsa_pow_pred(m, p)(y) = f(y)
            f(y) = rsa_pow_pred(m, p)(y)
        }
        rsa_pow_base(m, p)
        rsa_pow_pred(m, p)(Nat.0)
        f(Nat.0)
        forall(x: Nat) {
            if f(x) {
                rsa_pow_pred(m, p)(x)
                rsa_pow_step(m, p, x)
                rsa_pow_pred(m, p)(x.suc)
                f(x.suc)
            }
        }
        f(k)
        rsa_pow_pred(m, p)(k)
    }
}

/// RSA power congruence: for prime p, every exponent of the form k * (p - 1) + 1
/// behaves like exponent 1 modulo p.
theorem rsa_pow_congr(p: Nat, m: Nat, k: Nat) {
    p.is_prime implies m.pow(k * (p - Nat.1) + Nat.1).congr_mod(m, p)
} by {
    if p.is_prime {
        rsa_pow_run(p, m, k)
        rsa_pow_pred(m, p)(k)
    }
}

/// Helper: m * m^(p - 1) congruence rephrasing.
theorem fermat_euler_split(p: Nat, m: Nat) {
    p.is_prime
        implies (m * m.pow(p - Nat.1)).congr_mod(m * Nat.1, p)
} by {
    if p.is_prime {
        // m^p ≡ m (mod p) by Fermat's little theorem.
        fermats_little_congr(p, m)
        m.pow(p).congr_mod(m, p)
        // m * m^(p - 1) = m^p (using mul_pow_pred_eq_pow).
        mul_pow_pred_eq_pow(m, p)
        m * m.pow(p - Nat.1) = m.pow(p)
        (m * m.pow(p - Nat.1)).congr_mod(m, p)
        // Rewrite m as m * 1.
        m * Nat.1 = m
        (m * m.pow(p - Nat.1)).congr_mod(m * Nat.1, p)
    }
}

/// Fermat-Euler: for prime `p` and `m` coprime to `p`,
///   `m.pow(p - 1) ≡ 1  (mod p)`.
/// Equivalently, this is Euler's theorem at the prime `p` (since
/// `(p).totient = p - 1`). Cancels `m` from `m * m^(p - 1) ≡ m * 1 (mod p)`,
/// which holds by Fermat's little theorem.
theorem fermat_euler(p: Nat, m: Nat) {
    p.is_prime and m.coprime(p)
        implies m.pow(p - Nat.1).congr_mod(Nat.1, p)
} by {
    if p.is_prime and m.coprime(p) {
        fermat_euler_split(p, m)
        (m * m.pow(p - Nat.1)).congr_mod(m * Nat.1, p)
        cancel_coprime(m, p, m.pow(p - Nat.1), Nat.1)
    }
}
