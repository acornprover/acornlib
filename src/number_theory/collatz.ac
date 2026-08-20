from nat import Nat, div_lt, div_mul, lt_trans, lt_add_left, lt_add_suc, lt_suc, lt_imp_lt_suc
from graph import nat_odd, nat_odd_add
from analysis import iterate, iterate_zero, iterate_one, iterate_suc, iterate_suc_inner

numerals Nat

/// The Collatz map: `T(n) = 3n + 1` when `n` is odd, and `T(n) = n/2` when `n` is even.
define collatz(n: Nat) -> Nat {
    if nat_odd(n) {
        3 * n + 1
    } else {
        n.div(2)
    }
}

/// For odd inputs the Collatz map is `3n + 1`.
theorem collatz_odd_eq(n: Nat) {
    nat_odd(n) implies collatz(n) = 3 * n + 1
} by {
    if nat_odd(n) {
        collatz(n) = 3 * n + 1
    }
}

/// For even inputs the Collatz map is `n/2`.
theorem collatz_even_eq(n: Nat) {
    not nat_odd(n) implies collatz(n) = n.div(2)
} by {
    if not nat_odd(n) {
        collatz(n) = n.div(2)
    }
}

/// One is odd.
theorem collatz_one_odd {
    nat_odd(Nat.1) = true
}

/// Two is even.
theorem collatz_two_even {
    nat_odd(Nat.2) = false
}

/// Three is odd.
theorem collatz_three_odd {
    nat_odd(Nat.3) = true
}

/// Four is even.
theorem collatz_four_even {
    nat_odd(Nat.4) = false
}

/// Five is odd.
theorem collatz_five_odd {
    nat_odd(Nat.5) = true
}

/// Doubling an input makes it even.
theorem collatz_even_double(k: Nat) {
    nat_odd(Nat.2 * k) = false
} by {
    Nat.2 * k = k + k
    nat_odd_add(k, k)
    nat_odd(k + k) = (nat_odd(k) != nat_odd(k))
    (nat_odd(k) != nat_odd(k)) = false
    nat_odd(k + k) = false
    nat_odd(Nat.2 * k) = false
}

/// Eight is even.
theorem collatz_eight_even {
    nat_odd(Nat.8) = false
} by {
    collatz_even_double(Nat.4)
    nat_odd(Nat.2 * Nat.4) = false
    Nat.2 * Nat.4 = Nat.8
    nat_odd(Nat.8) = false
}

/// Ten is even.
theorem collatz_ten_even {
    nat_odd(Nat.10) = false
} by {
    collatz_even_double(Nat.5)
    nat_odd(Nat.2 * Nat.5) = false
    Nat.2 * Nat.5 = Nat.10
    nat_odd(Nat.10) = false
}

/// Sixteen is even.
theorem collatz_sixteen_even {
    nat_odd(Nat.16) = false
} by {
    collatz_even_double(Nat.8)
    nat_odd(Nat.2 * Nat.8) = false
    Nat.2 * Nat.8 = Nat.16
    nat_odd(Nat.16) = false
}

/// The Collatz map sends one to four: T(1) = 3·1 + 1 = 4.
theorem collatz_one {
    collatz(Nat.1) = Nat.4
} by {
    collatz_one_odd
    nat_odd(Nat.1) = true
    nat_odd(Nat.1)
    collatz_odd_eq(Nat.1)
    collatz(Nat.1) = 3 * Nat.1 + 1
    3 * Nat.1 + 1 = Nat.4
    collatz(Nat.1) = Nat.4
}

/// The Collatz map sends two to one: T(2) = 2/2 = 1.
theorem collatz_two {
    collatz(Nat.2) = Nat.1
} by {
    collatz_two_even
    nat_odd(Nat.2) = false
    not nat_odd(Nat.2)
    collatz_even_eq(Nat.2)
    collatz(Nat.2) = Nat.2.div(2)
    div_mul(Nat.1, Nat.2)
    (Nat.1 * Nat.2).div(Nat.2) = Nat.1
    Nat.1 * Nat.2 = Nat.2
    Nat.2.div(2) = Nat.1
    collatz(Nat.2) = Nat.1
}

/// The Collatz map sends four to two: T(4) = 4/2 = 2.
theorem collatz_four {
    collatz(Nat.4) = Nat.2
} by {
    collatz_four_even
    nat_odd(Nat.4) = false
    not nat_odd(Nat.4)
    collatz_even_eq(Nat.4)
    collatz(Nat.4) = Nat.4.div(2)
    div_mul(Nat.2, Nat.2)
    (Nat.2 * Nat.2).div(Nat.2) = Nat.2
    Nat.2 * Nat.2 = Nat.4
    Nat.4.div(2) = Nat.2
    collatz(Nat.4) = Nat.2
}

/// The Collatz map sends three to ten: T(3) = 3·3 + 1 = 10.
theorem collatz_three {
    collatz(Nat.3) = Nat.10
} by {
    collatz_three_odd
    nat_odd(Nat.3) = true
    nat_odd(Nat.3)
    collatz_odd_eq(Nat.3)
    collatz(Nat.3) = 3 * Nat.3 + 1
    3 * Nat.3 + 1 = Nat.10
    collatz(Nat.3) = Nat.10
}

/// The Collatz map sends ten to five: T(10) = 10/2 = 5.
theorem collatz_ten {
    collatz(Nat.10) = Nat.5
} by {
    collatz_ten_even
    nat_odd(Nat.10) = false
    not nat_odd(Nat.10)
    collatz_even_eq(Nat.10)
    collatz(Nat.10) = Nat.10.div(2)
    div_mul(Nat.5, Nat.2)
    (Nat.5 * Nat.2).div(Nat.2) = Nat.5
    Nat.5 * Nat.2 = Nat.10
    Nat.10.div(2) = Nat.5
    collatz(Nat.10) = Nat.5
}

/// The Collatz map sends five to sixteen: T(5) = 3·5 + 1 = 16.
theorem collatz_five {
    collatz(Nat.5) = Nat.16
} by {
    collatz_five_odd
    nat_odd(Nat.5) = true
    nat_odd(Nat.5)
    collatz_odd_eq(Nat.5)
    collatz(Nat.5) = 3 * Nat.5 + 1
    Nat.3 * Nat.5 = Nat.15
    Nat.15 + Nat.1 = Nat.16
    3 * Nat.5 + 1 = Nat.16
    collatz(Nat.5) = Nat.16
}

/// The Collatz map sends sixteen to eight: T(16) = 16/2 = 8.
theorem collatz_sixteen {
    collatz(Nat.16) = Nat.8
} by {
    collatz_sixteen_even
    nat_odd(Nat.16) = false
    not nat_odd(Nat.16)
    collatz_even_eq(Nat.16)
    collatz(Nat.16) = Nat.16.div(2)
    div_mul(Nat.8, Nat.2)
    (Nat.8 * Nat.2).div(Nat.2) = Nat.8
    Nat.8 * Nat.2 = Nat.16
    Nat.16.div(2) = Nat.8
    collatz(Nat.16) = Nat.8
}

/// The Collatz map sends eight to four: T(8) = 8/2 = 4.
theorem collatz_eight {
    collatz(Nat.8) = Nat.4
} by {
    collatz_eight_even
    nat_odd(Nat.8) = false
    not nat_odd(Nat.8)
    collatz_even_eq(Nat.8)
    collatz(Nat.8) = Nat.8.div(2)
    div_mul(Nat.4, Nat.2)
    (Nat.4 * Nat.2).div(Nat.2) = Nat.4
    Nat.4 * Nat.2 = Nat.8
    Nat.8.div(2) = Nat.4
    collatz(Nat.8) = Nat.4
}

/// The orbit of two visits the cycle: T⁴(2) = 1 along 2 → 1 → 4 → 2 → 1.
theorem collatz_orbit_two {
    collatz(collatz(collatz(collatz(Nat.2)))) = Nat.1
} by {
    collatz_two
    collatz(Nat.2) = Nat.1
    collatz(collatz(collatz(collatz(Nat.2)))) = collatz(collatz(collatz(Nat.1)))
    collatz_one
    collatz(Nat.1) = Nat.4
    collatz(collatz(collatz(collatz(Nat.2)))) = collatz(collatz(Nat.4))
    collatz_four
    collatz(Nat.4) = Nat.2
    collatz(collatz(collatz(collatz(Nat.2)))) = collatz(Nat.2)
    collatz_two
    collatz(Nat.2) = Nat.1
    collatz(collatz(collatz(collatz(Nat.2)))) = Nat.1
}

/// One lies on the cycle 1 → 4 → 2 → 1.
theorem collatz_one_cycle {
    iterate(collatz, Nat.3, Nat.1) = Nat.1
} by {
    iterate_suc(collatz, Nat.2, Nat.1)
    iterate(collatz, Nat.3, Nat.1) = collatz(iterate(collatz, Nat.2, Nat.1))
    iterate_suc(collatz, Nat.1, Nat.1)
    iterate(collatz, Nat.2, Nat.1) = collatz(iterate(collatz, Nat.1, Nat.1))
    iterate_one(collatz, Nat.1)
    iterate(collatz, Nat.1, Nat.1) = collatz(Nat.1)
    collatz_one
    collatz(Nat.1) = Nat.4
    iterate(collatz, Nat.1, Nat.1) = Nat.4
    iterate(collatz, Nat.2, Nat.1) = collatz(Nat.4)
    collatz_four
    collatz(Nat.4) = Nat.2
    iterate(collatz, Nat.2, Nat.1) = Nat.2
    iterate(collatz, Nat.3, Nat.1) = collatz(Nat.2)
    collatz_two
    collatz(Nat.2) = Nat.1
    iterate(collatz, Nat.3, Nat.1) = Nat.1
}

/// Two reaches one in four steps.
theorem collatz_two_reaches_one {
    iterate(collatz, Nat.4, Nat.2) = Nat.1
} by {
    iterate_suc_inner(collatz, Nat.3, Nat.2)
    iterate(collatz, Nat.4, Nat.2) = iterate(collatz, Nat.3, collatz(Nat.2))
    collatz_two
    collatz(Nat.2) = Nat.1
    iterate(collatz, Nat.4, Nat.2) = iterate(collatz, Nat.3, Nat.1)
    iterate_suc_inner(collatz, Nat.2, Nat.1)
    iterate(collatz, Nat.4, Nat.2) = iterate(collatz, Nat.2, collatz(Nat.1))
    collatz_one
    collatz(Nat.1) = Nat.4
    iterate(collatz, Nat.4, Nat.2) = iterate(collatz, Nat.2, Nat.4)
    iterate_suc_inner(collatz, Nat.1, Nat.4)
    iterate(collatz, Nat.4, Nat.2) = iterate(collatz, Nat.1, collatz(Nat.4))
    collatz_four
    collatz(Nat.4) = Nat.2
    iterate(collatz, Nat.4, Nat.2) = iterate(collatz, Nat.1, Nat.2)
    iterate_suc_inner(collatz, Nat.0, Nat.2)
    iterate(collatz, Nat.4, Nat.2) = iterate(collatz, Nat.0, collatz(Nat.2))
    collatz_two
    collatz(Nat.2) = Nat.1
    iterate(collatz, Nat.4, Nat.2) = iterate(collatz, Nat.0, Nat.1)
    iterate_zero(collatz, Nat.1)
    iterate(collatz, Nat.0, Nat.1) = Nat.1
    iterate(collatz, Nat.4, Nat.2) = Nat.1
}

/// Three reaches one in seven steps: 3 → 10 → 5 → 16 → 8 → 4 → 2 → 1.
theorem collatz_three_reaches_one {
    iterate(collatz, Nat.7, Nat.3) = Nat.1
} by {
    iterate_suc_inner(collatz, Nat.6, Nat.3)
    iterate(collatz, Nat.7, Nat.3) = iterate(collatz, Nat.6, collatz(Nat.3))
    collatz_three
    collatz(Nat.3) = Nat.10
    iterate(collatz, Nat.7, Nat.3) = iterate(collatz, Nat.6, Nat.10)
    iterate_suc_inner(collatz, Nat.5, Nat.10)
    iterate(collatz, Nat.7, Nat.3) = iterate(collatz, Nat.5, collatz(Nat.10))
    collatz_ten
    collatz(Nat.10) = Nat.5
    iterate(collatz, Nat.7, Nat.3) = iterate(collatz, Nat.5, Nat.5)
    iterate_suc_inner(collatz, Nat.4, Nat.5)
    iterate(collatz, Nat.7, Nat.3) = iterate(collatz, Nat.4, collatz(Nat.5))
    collatz_five
    collatz(Nat.5) = Nat.16
    iterate(collatz, Nat.7, Nat.3) = iterate(collatz, Nat.4, Nat.16)
    iterate_suc_inner(collatz, Nat.3, Nat.16)
    iterate(collatz, Nat.7, Nat.3) = iterate(collatz, Nat.3, collatz(Nat.16))
    collatz_sixteen
    collatz(Nat.16) = Nat.8
    iterate(collatz, Nat.7, Nat.3) = iterate(collatz, Nat.3, Nat.8)
    iterate_suc_inner(collatz, Nat.2, Nat.8)
    iterate(collatz, Nat.7, Nat.3) = iterate(collatz, Nat.2, collatz(Nat.8))
    collatz_eight
    collatz(Nat.8) = Nat.4
    iterate(collatz, Nat.7, Nat.3) = iterate(collatz, Nat.2, Nat.4)
    iterate_suc_inner(collatz, Nat.1, Nat.4)
    iterate(collatz, Nat.7, Nat.3) = iterate(collatz, Nat.1, collatz(Nat.4))
    collatz_four
    collatz(Nat.4) = Nat.2
    iterate(collatz, Nat.7, Nat.3) = iterate(collatz, Nat.1, Nat.2)
    iterate_suc_inner(collatz, Nat.0, Nat.2)
    iterate(collatz, Nat.7, Nat.3) = iterate(collatz, Nat.0, collatz(Nat.2))
    collatz_two
    collatz(Nat.2) = Nat.1
    iterate(collatz, Nat.7, Nat.3) = iterate(collatz, Nat.0, Nat.1)
    iterate_zero(collatz, Nat.1)
    iterate(collatz, Nat.0, Nat.1) = Nat.1
    iterate(collatz, Nat.7, Nat.3) = Nat.1
}

/// The Collatz map increases odd inputs greater than one: T(n) = 3n + 1 > n.
theorem collatz_odd_grows(n: Nat) {
    nat_odd(n) and Nat.1 < n implies n < collatz(n)
} by {
    if nat_odd(n) and Nat.1 < n {
        collatz_odd_eq(n)
        collatz(n) = 3 * n + 1
        Nat.0 < 2 * n + 1
        lt_add_left(n, Nat.0, 2 * n + 1)
        n + Nat.0 < n + (2 * n + 1)
        n < n + (2 * n + 1)
        n + (2 * n + 1) = 3 * n + 1
        n < 3 * n + 1
        n < collatz(n)
    }
}

/// The Collatz map decreases even inputs greater than two: T(n) = n/2 < n.
theorem collatz_even_shrinks(n: Nat) {
    not nat_odd(n) and Nat.2 < n implies collatz(n) < n
} by {
    if not nat_odd(n) and Nat.2 < n {
        collatz_even_eq(n)
        collatz(n) = n.div(2)
        Nat.0 < Nat.2
        lt_trans(Nat.0, Nat.2, n)
        Nat.0 < n
        n != Nat.0
        Nat.1 < Nat.2
        div_lt(n, Nat.2)
        n.div(2) < n
        collatz(n) < n
    }
}

/// Three grows under the Collatz map: T(3) = 10 > 3.
theorem collatz_three_grows {
    Nat.3 < collatz(Nat.3)
} by {
    collatz_three_odd
    nat_odd(Nat.3) = true
    nat_odd(Nat.3)
    lt_suc(Nat.1)
    Nat.1 < Nat.2
    lt_imp_lt_suc(Nat.1, Nat.2)
    Nat.1 < Nat.3
    collatz_odd_grows(Nat.3)
    nat_odd(Nat.3) and Nat.1 < Nat.3
    Nat.3 < collatz(Nat.3)
}

/// Four shrinks under the Collatz map: T(4) = 2 < 4.
theorem collatz_four_shrinks {
    collatz(Nat.4) < Nat.4
} by {
    collatz_four
    collatz(Nat.4) = Nat.2
    lt_add_suc(Nat.2, Nat.1)
    Nat.2 < Nat.4
    collatz(Nat.4) < Nat.4
}
