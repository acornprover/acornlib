from data.int.int_congruence import Int, int_congr_mod_symm, int_congr_mod_trans
from list import List
from nat import Nat, divides_self, divides_mul
from number_theory.crt_list import every_modulus_positive, system_modulus,
    system_modulus_cons
from pair import Pair, pair_new_first, pair_new_second
from zmod import int_mod_rel
numerals Nat
numerals Int

/// True if an integer belongs to the congruence class represented by a
/// `(modulus, residue)` pair.
define congruence_class_contains(cl: Pair[Nat, Nat], x: Int) -> Bool {
    int_mod_rel(cl.first, x, Int.from_nat(cl.second))
}

/// True if an integer belongs to at least one congruence class in a list.
define covers_int(system: List[Pair[Nat, Nat]], x: Int) -> Bool {
    match system {
        List.nil {
            false
        }
        List.cons(head, tail) {
            congruence_class_contains(head, x) or covers_int(tail, x)
        }
    }
}

/// The empty list covers no integer.
theorem covers_int_nil_false(x: Int) {
    not covers_int(List.nil[Pair[Nat, Nat]], x)
} by {
    covers_int(List.nil[Pair[Nat, Nat]], x) = false
}

/// Membership in a cons covering list unfolds to the head class or the tail.
theorem covers_int_cons_imp(head: Pair[Nat, Nat], tail: List[Pair[Nat, Nat]], x: Int) {
    covers_int(List.cons(head, tail), x)
        implies congruence_class_contains(head, x) or covers_int(tail, x)
} by {
    if covers_int(List.cons(head, tail), x) {
        congruence_class_contains(head, x) or covers_int(tail, x)
    }
}

/// The head class covers every integer it contains.
theorem covers_int_cons_left(head: Pair[Nat, Nat], tail: List[Pair[Nat, Nat]], x: Int) {
    congruence_class_contains(head, x) implies covers_int(List.cons(head, tail), x)
} by {
    if congruence_class_contains(head, x) {
        covers_int(List.cons(head, tail), x)
    }
}

/// A tail cover remains a cover after consing one more class.
theorem covers_int_cons_right(head: Pair[Nat, Nat], tail: List[Pair[Nat, Nat]], x: Int) {
    covers_int(tail, x) implies covers_int(List.cons(head, tail), x)
} by {
    if covers_int(tail, x) {
        covers_int(List.cons(head, tail), x)
    }
}

/// True if every integer is covered by the list of congruence classes.
define covers_all_int(system: List[Pair[Nat, Nat]]) -> Bool {
    forall(x: Int) {
        covers_int(system, x)
    }
}

/// A universal integer cover applies to any chosen integer.
theorem covers_all_int_apply(system: List[Pair[Nat, Nat]], x: Int) {
    covers_all_int(system) implies covers_int(system, x)
} by {
    if covers_all_int(system) {
        covers_all_int(system) = forall(y: Int) {
            covers_int(system, y)
        }
        forall(y: Int) {
            covers_int(system, y)
        }
        covers_int(system, x)
    }
}

/// A pointwise cover of every integer is a universal integer cover.
theorem covers_all_int_intro(system: List[Pair[Nat, Nat]]) {
    (forall(x: Int) { covers_int(system, x) }) implies covers_all_int(system)
} by {
    if forall(x: Int) { covers_int(system, x) } {
        covers_all_int(system) = forall(x: Int) {
            covers_int(system, x)
        }
        covers_all_int(system)
    }
}

/// A covering system is a finite list of positive-modulus congruence classes
/// whose union is all of the integers.
define is_covering_system(system: List[Pair[Nat, Nat]]) -> Bool {
    every_modulus_positive(system) and covers_all_int(system)
}

/// A covering system has positive moduli.
theorem covering_system_moduli_positive(system: List[Pair[Nat, Nat]]) {
    is_covering_system(system) implies every_modulus_positive(system)
} by {
    if is_covering_system(system) {
        every_modulus_positive(system)
    }
}

/// A covering system covers every integer.
theorem covering_system_covers_int(system: List[Pair[Nat, Nat]], x: Int) {
    is_covering_system(system) implies covers_int(system, x)
} by {
    if is_covering_system(system) {
        covers_all_int(system)
        covers_all_int_apply(system, x)
        covers_int(system, x)
    }
}

/// Adding one positive congruence class to a covering system preserves coverage.
theorem covering_system_cons_of_tail(head: Pair[Nat, Nat], tail: List[Pair[Nat, Nat]]) {
    head.first != Nat.0 and is_covering_system(tail)
        implies is_covering_system(List.cons(head, tail))
} by {
    if head.first != Nat.0 and is_covering_system(tail) {
        every_modulus_positive(tail)
        every_modulus_positive(List.cons(head, tail))
        covers_all_int(tail)
        forall(x: Int) {
            covers_all_int_apply(tail, x)
            covers_int(tail, x)
            covers_int_cons_right(head, tail, x)
            covers_int(List.cons(head, tail), x)
        }
        covers_all_int_intro(List.cons(head, tail))
        covers_all_int(List.cons(head, tail))
        is_covering_system(List.cons(head, tail))
    }
}

/// Congruence modulo one is the universal integer relation.
theorem int_mod_rel_one(x: Int, y: Int) {
    int_mod_rel(Nat.1, x, y)
} by {
    let diff: Int = x - y
    diff * Int.1 = diff
    Int.1.divides(diff)
    Int.from_nat(Nat.1) = Int.1
    Int.from_nat(Nat.1).divides(diff)
    Int.from_nat(Nat.1).divides(x - y)
}

/// Integer congruence is symmetric.
theorem int_mod_rel_symm(m: Nat, x: Int, y: Int) {
    int_mod_rel(m, x, y) implies int_mod_rel(m, y, x)
} by {
    if int_mod_rel(m, x, y) {
        x.congr_mod(y, m)
        int_congr_mod_symm(x, y, m)
        y.congr_mod(x, m)
        int_mod_rel(m, y, x)
    }
}

/// Integer congruence is transitive.
theorem int_mod_rel_trans(m: Nat, x: Int, y: Int, z: Int) {
    int_mod_rel(m, x, y) and int_mod_rel(m, y, z) implies int_mod_rel(m, x, z)
} by {
    if int_mod_rel(m, x, y) and int_mod_rel(m, y, z) {
        x.congr_mod(y, m)
        y.congr_mod(z, m)
        int_congr_mod_trans(x, y, z, m)
        x.congr_mod(z, m)
        int_mod_rel(m, x, z)
    }
}

/// A congruence class is closed under congruence modulo its modulus.
theorem congruence_class_contains_of_congruent(cl: Pair[Nat, Nat], x: Int, y: Int) {
    int_mod_rel(cl.first, x, y) and congruence_class_contains(cl, y)
        implies congruence_class_contains(cl, x)
} by {
    if int_mod_rel(cl.first, x, y) and congruence_class_contains(cl, y) {
        int_mod_rel(cl.first, y, Int.from_nat(cl.second))
        int_mod_rel_trans(cl.first, x, y, Int.from_nat(cl.second))
        int_mod_rel(cl.first, x, Int.from_nat(cl.second))
        congruence_class_contains(cl, x)
    }
}

/// The congruence class modulo one contains every integer.
theorem congruence_class_mod_one_contains(r: Nat, x: Int) {
    congruence_class_contains(Pair.new(Nat.1, r), x)
} by {
    pair_new_first[Nat, Nat](Nat.1, r)
    pair_new_second[Nat, Nat](Nat.1, r)
    Pair.new(Nat.1, r).first = Nat.1
    Pair.new(Nat.1, r).second = r
    int_mod_rel_one(x, Int.from_nat(r))
    int_mod_rel(Pair.new(Nat.1, r).first, x, Int.from_nat(Pair.new(Nat.1, r).second))
    congruence_class_contains(Pair.new(Nat.1, r), x)
}

/// A single congruence class modulo one is a covering system.
theorem singleton_mod_one_covering_system(r: Nat) {
    is_covering_system(List.cons(Pair.new(Nat.1, r), List.nil[Pair[Nat, Nat]]))
} by {
    let head: Pair[Nat, Nat] = Pair.new(Nat.1, r)
    let tail: List[Pair[Nat, Nat]] = List.nil[Pair[Nat, Nat]]
    pair_new_first[Nat, Nat](Nat.1, r)
    head.first = Nat.1
    head.first != Nat.0
    every_modulus_positive(List.cons(head, tail))
    forall(x: Int) {
        congruence_class_mod_one_contains(r, x)
        congruence_class_contains(head, x)
        covers_int_cons_left(head, tail, x)
        covers_int(List.cons(head, tail), x)
    }
    covers_all_int_intro(List.cons(head, tail))
    covers_all_int(List.cons(head, tail))
    is_covering_system(List.cons(head, tail))
}

/// The head modulus divides the combined modulus of a cons system.
theorem head_modulus_divides_cons_modulus(head: Pair[Nat, Nat], tail: List[Pair[Nat, Nat]]) {
    head.first.divides(system_modulus(List.cons(head, tail)))
} by {
    system_modulus_cons(head, tail)
    let big_mod: Nat = head.first * system_modulus(tail)
    divides_self(head.first)
    divides_mul(head.first, system_modulus(tail), head.first)
    head.first.divides(big_mod)
    head.first.divides(system_modulus(List.cons(head, tail)))
}

/// The tail combined modulus divides the combined modulus of a cons system.
theorem tail_modulus_divides_cons_modulus(head: Pair[Nat, Nat], tail: List[Pair[Nat, Nat]]) {
    system_modulus(tail).divides(system_modulus(List.cons(head, tail)))
} by {
    system_modulus_cons(head, tail)
    let tail_mod: Nat = system_modulus(tail)
    let big_mod: Nat = head.first * tail_mod
    divides_self(tail_mod)
    divides_mul(tail_mod, head.first, tail_mod)
    tail_mod.divides(tail_mod * head.first)
    tail_mod * head.first = head.first * tail_mod
    tail_mod.divides(big_mod)
    tail_mod.divides(system_modulus(List.cons(head, tail)))
}

/// True if every residue below the combined system modulus is covered.
define covers_all_residues(system: List[Pair[Nat, Nat]]) -> Bool {
    forall(r: Nat) {
        r < system_modulus(system) implies covers_int(system, Int.from_nat(r))
    }
}

/// Pointwise residue coverage gives the finite residue-check predicate.
theorem covers_all_residues_intro(system: List[Pair[Nat, Nat]]) {
    (forall(r: Nat) {
        r < system_modulus(system) implies covers_int(system, Int.from_nat(r))
    }) implies covers_all_residues(system)
} by {
    if forall(r: Nat) {
        r < system_modulus(system) implies covers_int(system, Int.from_nat(r))
    } {
        covers_all_residues(system) = forall(r: Nat) {
            r < system_modulus(system) implies covers_int(system, Int.from_nat(r))
        }
        covers_all_residues(system)
    }
}

/// The finite residue-check predicate applies to each residue below the
/// combined modulus.
theorem covers_all_residues_apply(system: List[Pair[Nat, Nat]], r: Nat) {
    covers_all_residues(system) and r < system_modulus(system)
        implies covers_int(system, Int.from_nat(r))
} by {
    if covers_all_residues(system) and r < system_modulus(system) {
        if not covers_int(system, Int.from_nat(r)) {
            not (r < system_modulus(system) implies covers_int(system, Int.from_nat(r)))
            not forall(s: Nat) {
                s < system_modulus(system) implies covers_int(system, Int.from_nat(s))
            }
            covers_all_residues(system) = forall(s: Nat) {
                s < system_modulus(system) implies covers_int(system, Int.from_nat(s))
            }
            not covers_all_residues(system)
            false
        }
        covers_int(system, Int.from_nat(r))
    }
}

/// Every covering system covers all residues below its combined modulus.
theorem covering_system_covers_all_residues(system: List[Pair[Nat, Nat]]) {
    is_covering_system(system) implies covers_all_residues(system)
} by {
    if is_covering_system(system) {
        covers_all_int(system)
        forall(r: Nat) {
            if r < system_modulus(system) {
                covers_all_int_apply(system, Int.from_nat(r))
                covers_int(system, Int.from_nat(r))
            }
        }
        covers_all_residues_intro(system)
        covers_all_residues(system)
    }
}
