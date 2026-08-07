from number_theory.coprime import Nat, coprime_divides_of_divides_mul
from nat import gcd_of_prime, gcd_zero_left, gcd_zero_right,
    gcd_divides_left, gcd_divides_right, gcd_nonzero_left, gcd_nonzero_right,
    gcd_comm, divides_gcd
from nat import gcd_mul_lcm
from number_theory.lcm import lcm_divides_left, lcm_divides_right
from nat import exp_zero, exp_ne_zero
from nat import divides_self, divides_zero, has_prime_divisor, strong_induction,
    true_below, divides_trans, mul_cancel_left, mul_to_zero, divides_mul_left,
    add_cancels_left, lte_antisymm, lte_ref, lt_suc_left, add_identity_right,
    alt_add_zero, lte_imp_not_lt, alt_suc_ne_zero, only_zero_lte_zero,
    nat_lte_meet_of_bounds
from list import List, list_not_contains_impl_count_zero
from list import product
from list import is_permutation, remove_one_count_self,
    remove_one_count_other, remove_one_cons_eq, remove_one_cons_neq,
    nil_count_zero, product_remove_one, count_append, product_append,
    permutation_preserves_product
from data.list.list_filter_count import filter_cons_of_true, filter_cons_of_false,
    filter_count_of_true, filter_count_of_false
numerals Nat

/// Helper: if a prime is the product of two factors, one factor is 1 and the
/// other equals the prime.
theorem prime_factor_dichotomy(p: Nat, d: Nat, k: Nat) {
    p.is_prime and d * k = p implies (d = Nat.1 and k = p) or (d = p and k = Nat.1)
} by {
    if p.is_prime and d * k = p {
        Nat.1 < p
        p != Nat.0
        // Neither factor can be 0 (their product is nonzero p).
        if d = Nat.0 {
            Nat.0 * k = p
            p = Nat.0
            false
        }
        if k = Nat.0 {
            d * Nat.0 = p
            p = Nat.0
            false
        }
        d != Nat.0
        k != Nat.0
        // If both > 1, p is composite.
        if Nat.1 < d and Nat.1 < k {
            exists(b: Nat, c: Nat) {
                Nat.1 < b and Nat.1 < c and p = b * c
            }
            p.is_composite
            not p.is_prime
            false
        }
        // So at least one of d, k equals 1.
        not (Nat.1 < d and Nat.1 < k)
        not (Nat.1 < d) or not (Nat.1 < k)
        Nat.1 <= d
        Nat.1 <= k
        if not (Nat.1 < d) {
            not (Nat.1 < d)
            d <= Nat.1
            d = Nat.1
        }
        if not (Nat.1 < k) {
            not (Nat.1 < k)
            k <= Nat.1
            k = Nat.1
        }
        d = Nat.1 or k = Nat.1
        if d = Nat.1 {
            Nat.1 * k = p
            k = p
            d = Nat.1 and k = p
            (d = Nat.1 and k = p) or (d = p and k = Nat.1)
        } else {
            k = Nat.1
            d * Nat.1 = p
            d = p
            d = p and k = Nat.1
            (d = Nat.1 and k = p) or (d = p and k = Nat.1)
        }
    }
}

/// Only 1 and a prime divide that prime.
theorem prime_divisor_is_one_or_self(p: Nat, d: Nat) {
    p.is_prime and d.divides(p) implies d = Nat.1 or d = p
} by {
    if p.is_prime and d.divides(p) {
        let k: Nat satisfy { d * k = p }
        prime_factor_dichotomy(p, d, k)
        (d = Nat.1 and k = p) or (d = p and k = Nat.1)
        d = Nat.1 or d = p
    }
}

/// Distinct primes are coprime.
theorem coprime_of_distinct_primes(p: Nat, q: Nat) {
    p.is_prime and q.is_prime and p != q implies p.coprime(q)
} by {
    if p.is_prime and q.is_prime and p != q {
        gcd_of_prime(p, q)
        if p.gcd(q) = Nat.1 {
            p.coprime(q)
        } else {
            p.divides(q)
            prime_divisor_is_one_or_self(q, p)
            p = Nat.1 or p = q
            Nat.1 < p
            p != Nat.1
            p = q
            false
        }
    }
}

/// True if every element of the list is a prime natural number.
define all_prime(list: List[Nat]) -> Bool {
    match list {
        List.nil {
            true
        }
        List.cons(head, tail) {
            head.is_prime and all_prime(tail)
        }
    }
}

/// The empty list is trivially all-prime.
theorem all_prime_nil {
    all_prime(List.nil[Nat])
}

/// Cons-introduction for all_prime.
theorem all_prime_cons_intro(head: Nat, tail: List[Nat]) {
    head.is_prime and all_prime(tail) implies all_prime(List.cons(head, tail))
} by {
    if head.is_prime and all_prime(tail) {
        all_prime(List.cons(head, tail))
    }
}

/// Cons-elimination for all_prime: a non-empty all-prime list has a prime head
/// and an all-prime tail.
theorem all_prime_cons_elim(head: Nat, tail: List[Nat]) {
    all_prime(List.cons(head, tail)) implies head.is_prime and all_prime(tail)
} by {
    if all_prime(List.cons(head, tail)) {
        head.is_prime and all_prime(tail)
    }
}

/// Existence of a prime factorisation: every positive natural is the product
/// of some list of primes.
theorem prime_factorisation_exists(n: Nat) {
    Nat.1 <= n implies exists(factors: List[Nat]) {
        product[Nat](factors) = n and all_prime(factors)
    }
} by {
    define f(m: Nat) -> Bool {
        Nat.1 <= m implies exists(factors: List[Nat]) {
            product[Nat](factors) = m and all_prime(factors)
        }
    }
    forall(k: Nat) {
        if true_below(f, k) {
            if Nat.1 <= k {
                if k = Nat.1 {
                    product[Nat](List.nil[Nat]) = Nat.1
                    product[Nat](List.nil[Nat]) = k
                    all_prime_nil
                    all_prime(List.nil[Nat])
                    exists(factors: List[Nat]) {
                        product[Nat](factors) = k and all_prime(factors)
                    }
                } else {
                    Nat.1 < k
                    has_prime_divisor(k)
                    let p: Nat satisfy { p.is_prime and p.divides(k) }
                    let q: Nat satisfy { p * q = k }
                    Nat.1 < p
                    // q != 0 (else k = 0).
                    if q = Nat.0 {
                        p * Nat.0 = Nat.0
                        k = Nat.0
                        false
                    }
                    q != Nat.0
                    Nat.1 <= q
                    // q < k since p > 1.
                    p * q = k
                    if q = k {
                        p * k = k
                        // p >= 2, so p * k = k means k = 0 (contradicting k >= 1).
                        let r: Nat satisfy { p = Nat.1 + r }
                        r != Nat.0
                        Nat.1 + r >= Nat.1.suc
                        p >= Nat.2
                        (Nat.1 + r) * k = k + r * k
                        p * k = k + r * k
                        k + r * k = k
                        r * k = Nat.0
                        k = Nat.0
                        false
                    }
                    q != k
                    q < k
                    // f(q) follows from true_below(f, k) and q < k; the
                    // explicit `let h` instantiates the universally
                    // quantified implication so the prover finds f(q).
                    let h: Bool = q < k implies f(q)
                    h
                    let qfactors: List[Nat] satisfy {
                        product[Nat](qfactors) = q and all_prime(qfactors)
                    }
                    let factors: List[Nat] = List.cons(p, qfactors)
                    product[Nat](factors) = p * product[Nat](qfactors)
                    p * product[Nat](qfactors) = p * q
                    p * q = k
                    product[Nat](factors) = k
                    all_prime_cons_intro(p, qfactors)
                    all_prime(factors)
                    exists(fs: List[Nat]) {
                        product[Nat](fs) = k and all_prime(fs)
                    }
                }
            }
            f(k)
        }
    }
    strong_induction(f)
    forall(m: Nat) { f(m) }
    f(n)
}

/// A canonical prime factorisation: a list of primes whose product is the
/// given natural. For zero the result is the empty list (a placeholder; the
/// universal property only constrains positive inputs).
let prime_factorisation(n: Nat) -> factors: List[Nat] satisfy {
    if Nat.1 <= n {
        product[Nat](factors) = n and all_prime(factors)
    } else {
        factors = List.nil[Nat]
    }
} by {
    if Nat.1 <= n {
        prime_factorisation_exists(n)
        let x: List[Nat] satisfy {
            product[Nat](x) = n and all_prime(x)
        }
    } else {
        let x: List[Nat] satisfy {
            x = List.nil[Nat]
        }
    }
}

/// The factorisation of a positive natural multiplies back to itself.
theorem prime_factorisation_product(n: Nat) {
    Nat.1 <= n implies product[Nat](prime_factorisation(n)) = n
}

/// Every entry in the factorisation of a positive natural is prime.
theorem prime_factorisation_all_prime(n: Nat) {
    Nat.1 <= n implies all_prime(prime_factorisation(n))
}

/// The factorisation of zero is the empty list (the placeholder choice).
theorem prime_factorisation_zero {
    prime_factorisation(Nat.0) = List.nil[Nat]
} by {
    not (Nat.1 <= Nat.0)
}

/// Forward direction of the coprime/no-shared-prime characterisation: if
/// a and b are coprime, no prime divides both.
theorem coprime_imp_no_shared_prime_factor(a: Nat, b: Nat, p: Nat) {
    a.coprime(b) and p.is_prime implies not (p.divides(a) and p.divides(b))
} by {
    if a.coprime(b) and p.is_prime {
        if p.divides(a) and p.divides(b) {
            // p divides gcd(a, b) = 1, but p > 1, contradiction.
            a.gcd(b) = Nat.1
            // gcd is the universal lower bound on common divisors.
            // Use the divides_gcd direction of the gcd universal property.
            // We need: p.divides(a.gcd(b)).
            // From divides_gcd in nat_gcd:
            //   d.divides(a) and d.divides(b) implies d.divides(a.gcd(b)).
            // (in this codebase the lemma name is divides_gcd_pair_converse
            // applied via divides_gcd; let's use divides_gcd directly).
            p.is_prime
            Nat.1 < p
            p != Nat.1
            // p.divides(a.gcd(b)) means p.divides(Nat.1).
            // p divides 1 means p = 1 (since 1's only divisor is 1).
            prime_divisor_is_one_or_self(p, p)
            // Hmm, need divides direction the other way.
            // Use: p.divides(a) and p.divides(b) imply p.divides(gcd(a, b))
            // Using gcd_divides as the universal property.
            let q: Nat satisfy { p * q = Nat.1 }
            // p * q = 1 with p > 1 forces both > 0 ... actually
            // p * q = 1 means q != 0, and p * q >= p * 1 = p > 1 unless q = 0.
            q != Nat.0
            Nat.1 <= q
            p * Nat.1 <= p * q
            p <= Nat.1
            false
        }
    }
}

/// Backward direction of the coprime/no-shared-prime characterisation: if a
/// and b are not coprime, some prime divides both.
theorem not_coprime_imp_shared_prime_factor(a: Nat, b: Nat) {
    not a.coprime(b) implies
        exists(p: Nat) { p.is_prime and p.divides(a) and p.divides(b) }
} by {
    if not a.coprime(b) {
        a.gcd(b) != Nat.1
        if a.gcd(b) = Nat.0 {
            // gcd = 0 implies a = 0 (from gcd_nonzero_left contrapositive).
            if a != Nat.0 {
                gcd_nonzero_left(a, b)
                a.gcd(b) != Nat.0
                false
            }
            a = Nat.0
            // Take any prime, e.g., 2; it divides 0 = a, and we need it to
            // divide b. b might not be 0, so pick a prime dividing b instead.
            // Actually if gcd(a, b) = 0 with a = 0, then gcd(0, b) = b, so b = 0.
            gcd_zero_left(b)
            Nat.0.gcd(b) = b
            b = Nat.0
            // Both zero: any prime works. Use has_prime_divisor on 2.
            // Actually simpler: pick a prime via has_prime_divisor on a value > 1.
            // For example, 2 has a prime divisor.
            Nat.1 < Nat.2
            has_prime_divisor(Nat.2)
            let q: Nat satisfy { q.is_prime and q.divides(Nat.2) }
            divides_zero(q)
            q.divides(a)
            q.divides(b)
            exists(p: Nat) { p.is_prime and p.divides(a) and p.divides(b) }
        } else {
            // gcd > 0 and gcd != 1, so gcd > 1.
            a.gcd(b) != Nat.0
            a.gcd(b) >= Nat.1
            a.gcd(b) > Nat.1
            Nat.1 < a.gcd(b)
            has_prime_divisor(a.gcd(b))
            let p: Nat satisfy { p.is_prime and p.divides(a.gcd(b)) }
            gcd_divides_left(a, b)
            divides_trans(p, a.gcd(b), a)
            p.divides(a)
            gcd_divides_right(a, b)
            divides_trans(p, a.gcd(b), b)
            p.divides(b)
            exists(q: Nat) { q.is_prime and q.divides(a) and q.divides(b) }
        }
    }
}

/// Coprimality is exactly the absence of shared prime factors.
theorem coprime_iff_no_shared_prime_factor(a: Nat, b: Nat) {
    a.coprime(b) = (forall(p: Nat) {
        p.is_prime implies not (p.divides(a) and p.divides(b))
    })
} by {
    if a.coprime(b) {
        forall(p: Nat) {
            coprime_imp_no_shared_prime_factor(a, b, p)
        }
    }
    if forall(p: Nat) {
        p.is_prime implies not (p.divides(a) and p.divides(b))
    } {
        if not a.coprime(b) {
            not_coprime_imp_shared_prime_factor(a, b)
            let p: Nat satisfy {
                p.is_prime and p.divides(a) and p.divides(b)
            }
            false
        }
    }
}

/// Forward direction (the universal part): a prime has no proper divisor.
theorem prime_imp_no_proper_divisor_at(n: Nat, k: Nat) {
    n.is_prime and Nat.1 < k and k < n implies not k.divides(n)
} by {
    if n.is_prime and Nat.1 < k and k < n {
        if k.divides(n) {
            prime_divisor_is_one_or_self(n, k)
            k = Nat.1 or k = n
            Nat.1 < k
            k != Nat.1
            k = n
            k != n
            false
        }
    }
}

/// Forward direction (packaged): a prime has no proper divisor strictly between 1 and itself.
theorem prime_imp_no_proper_divisor(n: Nat) {
    n.is_prime implies Nat.1 < n and forall(k: Nat) {
        Nat.1 < k and k < n implies not k.divides(n)
    }
} by {
    if n.is_prime {
        Nat.1 < n
        forall(k: Nat) {
            prime_imp_no_proper_divisor_at(n, k)
            Nat.1 < k and k < n implies not k.divides(n)
        }
        Nat.1 < n and forall(k: Nat) {
            Nat.1 < k and k < n implies not k.divides(n)
        }
    }
}

/// Backward direction: if n > 1 and no number strictly between 1 and n divides n, then n is prime.
theorem no_proper_divisor_imp_prime(n: Nat) {
    Nat.1 < n and (forall(k: Nat) {
        Nat.1 < k and k < n implies not k.divides(n)
    }) implies n.is_prime
} by {
    if Nat.1 < n and (forall(k: Nat) {
        Nat.1 < k and k < n implies not k.divides(n)
    }) {
        if not n.is_prime {
            Nat.1 < n
            n.is_composite
            let (b: Nat, c: Nat) satisfy {
                Nat.1 < b and Nat.1 < c and n = b * c
            }
            Nat.1 < c
            Nat.2 <= c
            Nat.1 <= b
            b * Nat.2 <= b * c
            b * Nat.2 = b + b
            b + b <= n
            b < b + b
            b < n
            let h: Bool = (Nat.1 < b and b < n) implies not b.divides(n)
            h
            not b.divides(n)
            b * c = n
            b.divides(n)
            false
        }
    }
}

/// Trial-division primality test: a natural is prime iff it is greater than
/// one and has no proper divisor (no divisor strictly between 1 and itself).
theorem prime_iff_no_proper_divisor(n: Nat) {
    n.is_prime = (Nat.1 < n and forall(k: Nat) {
        Nat.1 < k and k < n implies not k.divides(n)
    })
} by {
    if n.is_prime {
        prime_imp_no_proper_divisor(n)
    }
    if Nat.1 < n and forall(k: Nat) {
        Nat.1 < k and k < n implies not k.divides(n)
    } {
        no_proper_divisor_imp_prime(n)
    }
}

/// Cons-step for `all_prime` under `remove_one` when the head matches.
theorem all_prime_remove_one_cons_eq(head: Nat, tail: List[Nat]) {
    all_prime(List.cons(head, tail)) implies all_prime(List.cons(head, tail).remove_one(head))
} by {
    if all_prime(List.cons(head, tail)) {
        all_prime_cons_elim(head, tail)
        all_prime(tail)
        remove_one_cons_eq[Nat](head, tail)
        List.cons(head, tail).remove_one(head) = tail
        all_prime(List.cons(head, tail).remove_one(head))
    }
}

/// Cons-step for `all_prime` under `remove_one` when the head does not match
/// and the tail's removal is already known to preserve all-primeness.
theorem all_prime_remove_one_cons_neq(head: Nat, tail: List[Nat], item: Nat) {
    all_prime(List.cons(head, tail)) and head != item and all_prime(tail.remove_one(item))
        implies all_prime(List.cons(head, tail).remove_one(item))
} by {
    if all_prime(List.cons(head, tail)) and head != item and all_prime(tail.remove_one(item)) {
        all_prime_cons_elim(head, tail)
        head.is_prime
        remove_one_cons_neq[Nat](head, tail, item)
        List.cons(head, tail).remove_one(item) = List.cons(head, tail.remove_one(item))
        all_prime_cons_intro(head, tail.remove_one(item))
        all_prime(List.cons(head, tail.remove_one(item)))
        all_prime(List.cons(head, tail).remove_one(item))
    }
}

/// Removing one occurrence of an element from an all-prime list keeps it
/// all-prime.
theorem all_prime_remove_one(list: List[Nat], item: Nat) {
    all_prime(list) implies all_prime(list.remove_one(item))
} by {
    define p(l: List[Nat]) -> Bool {
        all_prime(l) implies all_prime(l.remove_one(item))
    }
    p(List.nil[Nat])
    forall(h: Nat, t: List[Nat]) {
        if p(t) {
            if all_prime(List.cons(h, t)) {
                all_prime_cons_elim(h, t)
                all_prime(t)
                let tail_rm: Bool = all_prime(t.remove_one(item))
                tail_rm
                if h = item {
                    all_prime_remove_one_cons_eq(h, t)
                    all_prime(List.cons(h, t).remove_one(h))
                    all_prime(List.cons(h, t).remove_one(item))
                } else {
                    all_prime_remove_one_cons_neq(h, t, item)
                    all_prime(List.cons(h, t).remove_one(item))
                }
            }
            p(List.cons(h, t))
        }
    }
    List.induction(function(l: List[Nat]) { p(l) })
    forall(l: List[Nat]) { p(l) }
    p(list)
}

/// Filtering an all-prime list preserves the primality of every entry.
theorem all_prime_filter(list: List[Nat], pred: Nat -> Bool) {
    all_prime(list) implies all_prime(list.filter(pred))
} by {
    define f(items: List[Nat]) -> Bool {
        all_prime(items) implies all_prime(items.filter(pred))
    }
    f(List.nil[Nat])
    forall(head: Nat, tail: List[Nat]) {
        if f(tail) {
            if all_prime(List.cons(head, tail)) {
                all_prime_cons_elim(head, tail)
                head.is_prime
                all_prime(tail)
                all_prime(tail.filter(pred))
                if pred(head) {
                    filter_cons_of_true(head, tail, pred)
                    List.cons(head, tail).filter(pred) =
                        List.cons(head, tail.filter(pred))
                    all_prime_cons_intro(head, tail.filter(pred))
                    all_prime(List.cons(head, tail).filter(pred))
                } else {
                    filter_cons_of_false(head, tail, pred)
                    List.cons(head, tail).filter(pred) = tail.filter(pred)
                    all_prime(List.cons(head, tail).filter(pred))
                }
            }
            f(List.cons(head, tail))
        }
    }
    List.induction(function(items: List[Nat]) { f(items) })
    forall(items: List[Nat]) { f(items) }
    f(list)
}

/// The product of an all-prime list is nonzero.
theorem all_prime_product_nonzero(list: List[Nat]) {
    all_prime(list) implies product[Nat](list) != Nat.0
} by {
    define f(items: List[Nat]) -> Bool {
        all_prime(items) implies product[Nat](items) != Nat.0
    }
    all_prime_nil
    all_prime(List.nil[Nat])
    product[Nat](List.nil[Nat]) = Nat.1
    Nat.1 != Nat.0
    product[Nat](List.nil[Nat]) != Nat.0
    f(List.nil[Nat])
    forall(head: Nat, tail: List[Nat]) {
        if f(tail) {
            if all_prime(List.cons(head, tail)) {
                all_prime_cons_elim(head, tail)
                head.is_prime
                Nat.1 < head
                head != Nat.0
                all_prime(tail)
                product[Nat](tail) != Nat.0
                product[Nat](List.cons(head, tail)) = head * product[Nat](tail)
                if product[Nat](List.cons(head, tail)) = Nat.0 {
                    head * product[Nat](tail) = Nat.0
                    mul_to_zero(head, product[Nat](tail))
                    false
                }
                product[Nat](List.cons(head, tail)) != Nat.0
            }
            f(List.cons(head, tail))
        }
    }
    List.induction(function(items: List[Nat]) { f(items) })
    forall(items: List[Nat]) { f(items) }
    f(list)
}

/// A prime cannot divide one.
theorem prime_does_not_divide_one(p: Nat) {
    p.is_prime implies not p.divides(Nat.1)
} by {
    if p.is_prime {
        Nat.1 < p
        if p.divides(Nat.1) {
            let k: Nat satisfy { p * k = Nat.1 }
            if k = Nat.0 {
                p * Nat.0 = Nat.0
                Nat.0 = Nat.1
                false
            }
            k != Nat.0
            Nat.1 <= k
            p * Nat.1 <= p * k
            p <= Nat.1
            not Nat.1 < p
            false
        }
    }
}

/// An all-prime list whose product is one must be empty (since each prime is
/// at least two and so contributes a factor greater than one).
theorem all_prime_product_eq_one_imp_nil(list: List[Nat]) {
    all_prime(list) and product[Nat](list) = Nat.1 implies list = List.nil[Nat]
} by {
    if all_prime(list) and product[Nat](list) = Nat.1 {
        match list {
            List.nil {
                list = List.nil[Nat]
            }
            List.cons(head, tail) {
                head.is_prime
                product[Nat](List.cons(head, tail)) = head * product[Nat](tail)
                head * product[Nat](tail) = Nat.1
                head.divides(Nat.1)
                prime_does_not_divide_one(head)
                false
            }
        }
    }
}

/// Cons-step of Euclid's lemma for prime-divides-product on all-prime lists.
theorem prime_divides_product_cons_step(head: Nat, tail: List[Nat], p: Nat) {
    (p.divides(product[Nat](tail)) implies tail.contains(p)) and
        head.is_prime and p.is_prime and p.divides(head * product[Nat](tail))
        implies List.cons(head, tail).contains(p)
} by {
    if (p.divides(product[Nat](tail)) implies tail.contains(p)) and
        head.is_prime and p.is_prime and p.divides(head * product[Nat](tail)) {
        gcd_of_prime(p, head)
        if p.divides(head) {
            prime_divisor_is_one_or_self(head, p)
            p = Nat.1 or p = head
            Nat.1 < p
            p != Nat.1
            p = head
            head = p
            List.cons(head, tail).contains(p)
        } else {
            p.gcd(head) = Nat.1
            p.coprime(head)
            coprime_divides_of_divides_mul(p, head, product[Nat](tail))
            p.divides(product[Nat](tail))
            tail.contains(p)
            List.cons(head, tail).contains(p)
        }
    }
}

/// Euclid's lemma extended to lists: if a prime divides the product of an
/// all-prime list, then the prime occurs in the list.
theorem prime_divides_product_imp_contains(list: List[Nat], p: Nat) {
    all_prime(list) and p.is_prime and p.divides(product[Nat](list))
        implies list.contains(p)
} by {
    define f(l: List[Nat]) -> Bool {
        all_prime(l) and p.is_prime and p.divides(product[Nat](l))
            implies l.contains(p)
    }
    if all_prime(List.nil[Nat]) and p.is_prime and p.divides(product[Nat](List.nil[Nat])) {
        product[Nat](List.nil[Nat]) = Nat.1
        p.divides(Nat.1)
        prime_does_not_divide_one(p)
        false
    }
    f(List.nil[Nat])
    forall(h: Nat, t: List[Nat]) {
        if f(t) {
            if all_prime(List.cons(h, t)) and p.is_prime and p.divides(product[Nat](List.cons(h, t))) {
                all_prime_cons_elim(h, t)
                h.is_prime
                all_prime(t)
                product[Nat](List.cons(h, t)) = h * product[Nat](t)
                p.divides(h * product[Nat](t))
                let tail_implication: Bool = p.divides(product[Nat](t)) implies t.contains(p)
                tail_implication
                prime_divides_product_cons_step(h, t, p)
                List.cons(h, t).contains(p)
            }
            f(List.cons(h, t))
        }
    }
    List.induction(function(l: List[Nat]) { f(l) })
    forall(l: List[Nat]) { f(l) }
    f(list)
}

/// Uniqueness side of the fundamental theorem of arithmetic: any two prime
/// factorisations of the same natural number are permutations of each other.
theorem prime_factorisation_unique(l1: List[Nat], l2: List[Nat]) {
    all_prime(l1) and all_prime(l2) and product[Nat](l1) = product[Nat](l2)
        implies is_permutation(l1, l2)
} by {
    define p(first: List[Nat]) -> Bool {
        forall(other: List[Nat]) {
            all_prime(first) and all_prime(other) and product[Nat](first) = product[Nat](other)
                implies is_permutation(first, other)
        }
    }
    forall(other: List[Nat]) {
        if all_prime(List.nil[Nat]) and all_prime(other) and product[Nat](List.nil[Nat]) = product[Nat](other) {
            product[Nat](List.nil[Nat]) = Nat.1
            product[Nat](other) = Nat.1
            all_prime_product_eq_one_imp_nil(other)
            other = List.nil[Nat]
            forall(x: Nat) {
                nil_count_zero[Nat](x)
                List.nil[Nat].count(x) = other.count(x)
            }
            is_permutation(List.nil[Nat], other)
        }
    }
    p(List.nil[Nat])
    forall(head: Nat, tail: List[Nat]) {
        if p(tail) {
            forall(other: List[Nat]) {
                if all_prime(List.cons(head, tail)) and all_prime(other) and product[Nat](List.cons(head, tail)) = product[Nat](other) {
                    head.is_prime
                    Nat.1 < head
                    head != Nat.0
                    all_prime(tail)
                    product[Nat](List.cons(head, tail)) = head * product[Nat](tail)
                    head * product[Nat](tail) = product[Nat](other)
                    head.divides(product[Nat](other))
                    prime_divides_product_imp_contains(other, head)
                    other.contains(head)
                    product_remove_one[Nat](other, head)
                    head * product[Nat](other.remove_one(head)) = product[Nat](other)
                    head * product[Nat](other.remove_one(head)) = head * product[Nat](tail)
                    mul_cancel_left(head, product[Nat](other.remove_one(head)), product[Nat](tail))
                    product[Nat](other.remove_one(head)) = product[Nat](tail)
                    product[Nat](tail) = product[Nat](other.remove_one(head))
                    all_prime_remove_one(other, head)
                    all_prime(other.remove_one(head))
                    is_permutation(tail, other.remove_one(head))
                    remove_one_count_self(other, head)
                    other.remove_one(head).count(head) + Nat.1 = other.count(head)
                    forall(x: Nat) {
                        if x = head {
                            x = head
                            List.cons(head, tail).count(x) = Nat.1 + tail.count(x)
                            tail.count(x) = tail.count(head)
                            tail.count(head) = other.remove_one(head).count(head)
                            other.count(head) = other.count(x)
                            Nat.1 + tail.count(head) = other.remove_one(head).count(head) + Nat.1
                            List.cons(head, tail).count(x) = other.count(x)
                        } else {
                            head != x
                            List.cons(head, tail).count(x) = tail.count(x)
                            tail.count(x) = other.remove_one(head).count(x)
                            remove_one_count_other(other, head, x)
                            other.remove_one(head).count(x) = other.count(x)
                            List.cons(head, tail).count(x) = other.count(x)
                        }
                    }
                    is_permutation(List.cons(head, tail), other)
                }
            }
            p(List.cons(head, tail))
        }
    }
    List.induction(function(l: List[Nat]) { p(l) })
}

/// Concatenating two all-prime lists yields an all-prime list.
theorem all_prime_append(left: List[Nat], right: List[Nat]) {
    all_prime(left) and all_prime(right) implies all_prime(left + right)
} by {
    define p(l: List[Nat]) -> Bool {
        all_prime(l) and all_prime(right) implies all_prime(l + right)
    }
    List.nil[Nat] + right = right
    p(List.nil[Nat])
    forall(head: Nat, tail: List[Nat]) {
        if p(tail) {
            if all_prime(List.cons(head, tail)) and all_prime(right) {
                all_prime_cons_elim(head, tail)
                head.is_prime
                all_prime(tail)
                all_prime(tail + right)
                List.cons(head, tail) + right = List.cons(head, tail + right)
                all_prime_cons_intro(head, tail + right)
                all_prime(List.cons(head, tail + right))
                all_prime(List.cons(head, tail) + right)
            }
            p(List.cons(head, tail))
        }
    }
    List.induction(function(l: List[Nat]) { p(l) })
    forall(l: List[Nat]) { p(l) }
    p(left)
}

/// Every entry of an all-prime list is prime.
theorem all_prime_only_primes(list: List[Nat], x: Nat) {
    all_prime(list) and list.contains(x) implies x.is_prime
} by {
    define p(l: List[Nat]) -> Bool {
        all_prime(l) and l.contains(x) implies x.is_prime
    }
    not List.nil[Nat].contains(x)
    p(List.nil[Nat])
    forall(head: Nat, tail: List[Nat]) {
        if p(tail) {
            if all_prime(List.cons(head, tail)) and List.cons(head, tail).contains(x) {
                all_prime_cons_elim(head, tail)
                head.is_prime
                all_prime(tail)
                if head = x {
                    x.is_prime
                } else {
                    tail.contains(x)
                    x.is_prime
                }
            }
            p(List.cons(head, tail))
        }
    }
    List.induction(function(l: List[Nat]) { p(l) })
    forall(l: List[Nat]) { p(l) }
    p(list)
}

/// The multiplicity of a prime in the canonical factorisation of a natural.
/// For non-primes, for one, and for zero, this evaluates to zero (the factorisation
/// of zero is the empty placeholder).
define count_prime_factor(p: Nat, n: Nat) -> Nat {
    prime_factorisation(n).count(p)
}

/// The product of the prime factors of `n` selected by `pred`.
define selected_prime_factor_product(n: Nat, pred: Nat -> Bool) -> Nat {
    product[Nat](prime_factorisation(n).filter(pred))
}

/// Bridging lemma: any list witnessing a prime factorisation of a positive
/// natural has the same prime-counts as the canonical factorisation.
theorem count_prime_factor_invariant(p: Nat, n: Nat, list: List[Nat]) {
    Nat.1 <= n and all_prime(list) and product[Nat](list) = n
        implies list.count(p) = count_prime_factor(p, n)
} by {
    if Nat.1 <= n and all_prime(list) and product[Nat](list) = n {
        prime_factorisation_product(n)
        prime_factorisation_all_prime(n)
        product[Nat](prime_factorisation(n)) = n
        all_prime(prime_factorisation(n))
        product[Nat](list) = product[Nat](prime_factorisation(n))
        prime_factorisation_unique(list, prime_factorisation(n))
        is_permutation(list, prime_factorisation(n))
        list.count(p) = prime_factorisation(n).count(p)
        list.count(p) = count_prime_factor(p, n)
    }
}

/// A selected prime-factor product is nonzero.
theorem selected_prime_factor_product_nonzero(n: Nat, pred: Nat -> Bool) {
    n != Nat.0 implies selected_prime_factor_product(n, pred) != Nat.0
} by {
    if n != Nat.0 {
        prime_factorisation_all_prime(n)
        all_prime(prime_factorisation(n))
        all_prime_filter(prime_factorisation(n), pred)
        all_prime(prime_factorisation(n).filter(pred))
        all_prime_product_nonzero(prime_factorisation(n).filter(pred))
        product[Nat](prime_factorisation(n).filter(pred)) != Nat.0
        selected_prime_factor_product(n, pred) != Nat.0
    }
}

/// Selecting a prime preserves its multiplicity; rejecting it gives
/// multiplicity zero.
theorem count_prime_factor_selected_product(
    p: Nat, n: Nat, pred: Nat -> Bool
) {
    n != Nat.0 implies count_prime_factor(
        p, selected_prime_factor_product(n, pred)) =
            if pred(p) { count_prime_factor(p, n) } else { Nat.0 }
} by {
    if n != Nat.0 {
        let items = prime_factorisation(n).filter(pred)
        selected_prime_factor_product_nonzero(n, pred)
        selected_prime_factor_product(n, pred) != Nat.0
        Nat.1 <= selected_prime_factor_product(n, pred)
        prime_factorisation_all_prime(n)
        all_prime(prime_factorisation(n))
        all_prime_filter(prime_factorisation(n), pred)
        all_prime(items)
        product[Nat](items) = selected_prime_factor_product(n, pred)
        count_prime_factor_invariant(
            p, selected_prime_factor_product(n, pred), items)
        items.count(p) = count_prime_factor(
            p, selected_prime_factor_product(n, pred))
        if pred(p) {
            filter_count_of_true(prime_factorisation(n), pred, p)
            items.count(p) = prime_factorisation(n).count(p)
            items.count(p) = count_prime_factor(p, n)
            count_prime_factor(p, selected_prime_factor_product(n, pred)) =
                count_prime_factor(p, n)
        } else {
            filter_count_of_false(prime_factorisation(n), pred, p)
            items.count(p) = Nat.0
            count_prime_factor(p, selected_prime_factor_product(n, pred)) = Nat.0
        }
        count_prime_factor(p, selected_prime_factor_product(n, pred)) =
            if pred(p) { count_prime_factor(p, n) } else { Nat.0 }
    }
}

/// The factorisation of one is the empty list, so the prime count of one is
/// always zero.
theorem count_prime_factor_one(p: Nat) {
    count_prime_factor(p, Nat.1) = Nat.0
} by {
    Nat.1 <= Nat.1
    prime_factorisation_product(Nat.1)
    product[Nat](prime_factorisation(Nat.1)) = Nat.1
    prime_factorisation_all_prime(Nat.1)
    all_prime(prime_factorisation(Nat.1))
    all_prime_product_eq_one_imp_nil(prime_factorisation(Nat.1))
    prime_factorisation(Nat.1) = List.nil[Nat]
    nil_count_zero[Nat](p)
    prime_factorisation(Nat.1).count(p) = Nat.0
}

/// The prime count of a non-prime is always zero, since prime factorisations
/// only contain primes.
theorem count_prime_factor_non_prime(p: Nat, n: Nat) {
    not p.is_prime implies count_prime_factor(p, n) = Nat.0
} by {
    if not p.is_prime {
        if prime_factorisation(n).contains(p) {
            if Nat.1 <= n {
                prime_factorisation_all_prime(n)
                all_prime(prime_factorisation(n))
                all_prime_only_primes(prime_factorisation(n), p)
                p.is_prime
                false
            } else {
                prime_factorisation_zero
                n = Nat.0
                prime_factorisation(n) = List.nil[Nat]
                not List.nil[Nat].contains(p)
                false
            }
        }
        not prime_factorisation(n).contains(p)
        list_not_contains_impl_count_zero[Nat](prime_factorisation(n), p)
        prime_factorisation(n).count(p) = Nat.0
    }
}

/// The prime count of a prime in itself is one.
theorem count_prime_factor_self(p: Nat) {
    p.is_prime implies count_prime_factor(p, p) = Nat.1
} by {
    if p.is_prime {
        Nat.1 < p
        Nat.1 <= p
        all_prime_nil
        all_prime_cons_intro(p, List.nil[Nat])
        all_prime(List.cons(p, List.nil[Nat]))
        product[Nat](List.cons(p, List.nil[Nat])) = p * product[Nat](List.nil[Nat])
        product[Nat](List.nil[Nat]) = Nat.1
        product[Nat](List.cons(p, List.nil[Nat])) = p * Nat.1
        product[Nat](List.cons(p, List.nil[Nat])) = p
        count_prime_factor_invariant(p, p, List.cons(p, List.nil[Nat]))
        List.cons(p, List.nil[Nat]).count(p) = count_prime_factor(p, p)
        nil_count_zero[Nat](p)
        List.nil[Nat].count(p) = Nat.0
        List.cons(p, List.nil[Nat]).count(p) = Nat.1 + List.nil[Nat].count(p)
        List.cons(p, List.nil[Nat]).count(p) = Nat.1 + Nat.0
        Nat.1 + Nat.0 = Nat.1
        List.cons(p, List.nil[Nat]).count(p) = Nat.1
        count_prime_factor(p, p) = Nat.1
    }
}

/// Multiplicativity of the prime-multiplicity function: for nonzero a and b,
/// the prime count of a product splits as the sum of prime counts.
theorem count_prime_factor_mul(p: Nat, a: Nat, b: Nat) {
    a != Nat.0 and b != Nat.0 implies
        count_prime_factor(p, a * b) =
            count_prime_factor(p, a) + count_prime_factor(p, b)
} by {
    if a != Nat.0 and b != Nat.0 {
        Nat.1 <= a
        Nat.1 <= b
        let la: List[Nat] = prime_factorisation(a)
        let lb: List[Nat] = prime_factorisation(b)
        prime_factorisation_product(a)
        prime_factorisation_product(b)
        prime_factorisation_all_prime(a)
        prime_factorisation_all_prime(b)
        product[Nat](la) = a
        product[Nat](lb) = b
        all_prime(la)
        all_prime(lb)
        let combined: List[Nat] = la + lb
        all_prime_append(la, lb)
        all_prime(combined)
        product_append[Nat](la, lb)
        product[Nat](combined) = product[Nat](la) * product[Nat](lb)
        product[Nat](combined) = a * b
        if a * b = Nat.0 {
            mul_to_zero(a, b)
            a = Nat.0 or b = Nat.0
            false
        }
        a * b != Nat.0
        Nat.1 <= a * b
        count_prime_factor_invariant(p, a * b, combined)
        combined.count(p) = count_prime_factor(p, a * b)
        count_append[Nat](la, lb, p)
        combined.count(p) = la.count(p) + lb.count(p)
        la.count(p) = count_prime_factor(p, a)
        lb.count(p) = count_prime_factor(p, b)
        count_prime_factor(p, a * b) = count_prime_factor(p, a) + count_prime_factor(p, b)
    }
}

/// The prime count of a prime `q` at a different prime `p` is zero, since
/// `q`'s factorisation is a singleton.
theorem count_prime_factor_other_prime(p: Nat, q: Nat) {
    p.is_prime and q.is_prime and p != q implies count_prime_factor(p, q) = Nat.0
} by {
    if p.is_prime and q.is_prime and p != q {
        Nat.1 < q
        Nat.1 <= q
        all_prime_nil
        all_prime_cons_intro(q, List.nil[Nat])
        all_prime(List.cons(q, List.nil[Nat]))
        product[Nat](List.cons(q, List.nil[Nat])) = q * product[Nat](List.nil[Nat])
        product[Nat](List.nil[Nat]) = Nat.1
        product[Nat](List.cons(q, List.nil[Nat])) = q * Nat.1
        product[Nat](List.cons(q, List.nil[Nat])) = q
        count_prime_factor_invariant(p, q, List.cons(q, List.nil[Nat]))
        List.cons(q, List.nil[Nat]).count(p) = count_prime_factor(p, q)
        q != p
        nil_count_zero[Nat](p)
        List.nil[Nat].count(p) = Nat.0
        List.cons(q, List.nil[Nat]).count(p) = List.nil[Nat].count(p)
        List.cons(q, List.nil[Nat]).count(p) = Nat.0
        count_prime_factor(p, q) = Nat.0
    }
}

/// Forward direction of the divisibility characterisation: if `a` divides
/// nonzero `b`, then every prime count of `a` is bounded by the matching
/// count of `b`.
theorem divides_imp_count_prime_factor_le(p: Nat, a: Nat, b: Nat) {
    a != Nat.0 and b != Nat.0 and a.divides(b)
        implies count_prime_factor(p, a) <= count_prime_factor(p, b)
} by {
    if a != Nat.0 and b != Nat.0 and a.divides(b) {
        let c: Nat satisfy { a * c = b }
        if c = Nat.0 {
            a * Nat.0 = Nat.0
            b = Nat.0
            false
        }
        c != Nat.0
        count_prime_factor_mul(p, a, c)
        count_prime_factor(p, a * c) = count_prime_factor(p, a) + count_prime_factor(p, c)
        count_prime_factor(p, b) = count_prime_factor(p, a) + count_prime_factor(p, c)
        count_prime_factor(p, a) <= count_prime_factor(p, b)
    }
}

/// Two positive naturals with identical prime-multiplicity profiles are
/// equal. The non-prime case is handled automatically since prime
/// factorisations only contain primes.
theorem count_prime_factor_ext(a: Nat, b: Nat) {
    a != Nat.0 and b != Nat.0 and
        (forall(p: Nat) {
            p.is_prime implies count_prime_factor(p, a) = count_prime_factor(p, b)
        })
        implies a = b
} by {
    if a != Nat.0 and b != Nat.0 and
        (forall(p: Nat) {
            p.is_prime implies count_prime_factor(p, a) = count_prime_factor(p, b)
        }) {
        Nat.1 <= a
        Nat.1 <= b
        let la: List[Nat] = prime_factorisation(a)
        let lb: List[Nat] = prime_factorisation(b)
        prime_factorisation_product(a)
        prime_factorisation_product(b)
        prime_factorisation_all_prime(a)
        prime_factorisation_all_prime(b)
        product[Nat](la) = a
        product[Nat](lb) = b
        all_prime(la)
        all_prime(lb)
        forall(x: Nat) {
            if x.is_prime {
                count_prime_factor(x, a) = count_prime_factor(x, b)
                la.count(x) = lb.count(x)
            } else {
                count_prime_factor_non_prime(x, a)
                count_prime_factor_non_prime(x, b)
                count_prime_factor(x, a) = Nat.0
                count_prime_factor(x, b) = Nat.0
                la.count(x) = Nat.0
                lb.count(x) = Nat.0
                la.count(x) = lb.count(x)
            }
        }
        is_permutation(la, lb)
        permutation_preserves_product[Nat](la, lb)
        product[Nat](la) = product[Nat](lb)
        a = b
    }
}

/// Backward direction induction step: a prime head with a sufficient count in
/// `b` divides `b` and bounds the per-prime count of the cofactor.
theorem count_divisor_quotient(head: Nat, b: Nat) {
    head.is_prime and b != Nat.0 and Nat.1 <= count_prime_factor(head, b)
        implies exists(q: Nat) {
            q != Nat.0 and head * q = b
        }
} by {
    if head.is_prime and b != Nat.0 and Nat.1 <= count_prime_factor(head, b) {
        Nat.1 <= b
        prime_factorisation_all_prime(b)
        prime_factorisation_product(b)
        all_prime(prime_factorisation(b))
        product[Nat](prime_factorisation(b)) = b
        prime_factorisation(b).count(head) >= Nat.1
        if not prime_factorisation(b).contains(head) {
            list_not_contains_impl_count_zero[Nat](prime_factorisation(b), head)
            prime_factorisation(b).count(head) = Nat.0
            Nat.1 <= Nat.0
            false
        }
        prime_factorisation(b).contains(head)
        product_remove_one[Nat](prime_factorisation(b), head)
        head * product[Nat](prime_factorisation(b).remove_one(head)) = product[Nat](prime_factorisation(b))
        head * product[Nat](prime_factorisation(b).remove_one(head)) = b
        let q: Nat = product[Nat](prime_factorisation(b).remove_one(head))
        head * q = b
        if q = Nat.0 {
            head * Nat.0 = Nat.0
            b = Nat.0
            false
        }
        q != Nat.0
        exists(qq: Nat) { qq != Nat.0 and head * qq = b }
    }
}

/// Backward direction (cons step): if dropping the head from the count
/// constraint already gives a divisibility, multiplying back by the head
/// extends the divisibility to the full product.
theorem product_all_prime_divides_step(head: Nat, tail: List[Nat], b: Nat) {
    head.is_prime and all_prime(tail) and b != Nat.0 and
    (forall(p: Nat) {
        p.is_prime implies List.cons(head, tail).count(p) <= count_prime_factor(p, b)
    }) and
    (forall(b2: Nat) {
        b2 != Nat.0 and (forall(p: Nat) {
            p.is_prime implies tail.count(p) <= count_prime_factor(p, b2)
        }) implies product[Nat](tail).divides(b2)
    })
    implies product[Nat](List.cons(head, tail)).divides(b)
} by {
    if head.is_prime and all_prime(tail) and b != Nat.0 and
        (forall(p: Nat) {
            p.is_prime implies List.cons(head, tail).count(p) <= count_prime_factor(p, b)
        }) and
        (forall(b2: Nat) {
            b2 != Nat.0 and (forall(p: Nat) {
                p.is_prime implies tail.count(p) <= count_prime_factor(p, b2)
            }) implies product[Nat](tail).divides(b2)
        }) {
        head != Nat.0
        List.cons(head, tail).count(head) = Nat.1 + tail.count(head)
        List.cons(head, tail).count(head) >= Nat.1
        List.cons(head, tail).count(head) <= count_prime_factor(head, b)
        Nat.1 <= count_prime_factor(head, b)
        count_divisor_quotient(head, b)
        let q: Nat satisfy { q != Nat.0 and head * q = b }
        forall(p: Nat) {
            if p.is_prime {
                count_prime_factor_mul(p, head, q)
                count_prime_factor(p, head * q) =
                    count_prime_factor(p, head) + count_prime_factor(p, q)
                count_prime_factor(p, b) =
                    count_prime_factor(p, head) + count_prime_factor(p, q)
                List.cons(head, tail).count(p) <= count_prime_factor(p, b)
                if p = head {
                    count_prime_factor_self(head)
                    count_prime_factor(head, head) = Nat.1
                    count_prime_factor(p, head) = Nat.1
                    Nat.1 + count_prime_factor(p, q) = count_prime_factor(p, b)
                    List.cons(head, tail).count(p) = Nat.1 + tail.count(head)
                    p = head
                    tail.count(p) = tail.count(head)
                    Nat.1 + tail.count(p) <= Nat.1 + count_prime_factor(p, q)
                    tail.count(p) <= count_prime_factor(p, q)
                } else {
                    head != p
                    count_prime_factor_other_prime(p, head)
                    count_prime_factor(p, head) = Nat.0
                    count_prime_factor(p, q) = count_prime_factor(p, b)
                    List.cons(head, tail).count(p) = tail.count(p)
                    tail.count(p) <= count_prime_factor(p, q)
                }
            }
        }
        if q != Nat.0 and (forall(p: Nat) {
            p.is_prime implies tail.count(p) <= count_prime_factor(p, q)
        }) {
            product[Nat](tail).divides(q)
        }
        product[Nat](tail).divides(q)
        divides_mul_left(product[Nat](tail), q, head)
        (head * product[Nat](tail)).divides(head * q)
        (head * product[Nat](tail)).divides(b)
        product[Nat](List.cons(head, tail)) = head * product[Nat](tail)
        product[Nat](List.cons(head, tail)).divides(b)
    }
}

/// Backward direction (helper): an all-prime list whose count is bounded by
/// the prime profile of nonzero `b` has a product dividing `b`.
theorem product_all_prime_divides(la: List[Nat]) {
    all_prime(la) implies forall(b: Nat) {
        b != Nat.0 and (forall(p: Nat) {
            p.is_prime implies la.count(p) <= count_prime_factor(p, b)
        }) implies product[Nat](la).divides(b)
    }
} by {
    define q(l: List[Nat]) -> Bool {
        all_prime(l) implies forall(b: Nat) {
            b != Nat.0 and (forall(p: Nat) {
                p.is_prime implies l.count(p) <= count_prime_factor(p, b)
            }) implies product[Nat](l).divides(b)
        }
    }
    if all_prime(List.nil[Nat]) {
        forall(b: Nat) {
            if b != Nat.0 and (forall(p: Nat) {
                p.is_prime implies List.nil[Nat].count(p) <= count_prime_factor(p, b)
            }) {
                product[Nat](List.nil[Nat]) = Nat.1
                Nat.1.divides(b)
                product[Nat](List.nil[Nat]).divides(b)
            }
        }
    }
    q(List.nil[Nat])
    forall(head: Nat, tail: List[Nat]) {
        if q(tail) {
            if all_prime(List.cons(head, tail)) {
                all_prime_cons_elim(head, tail)
                head.is_prime
                all_prime(tail)
                let tail_ih: Bool = forall(b2: Nat) {
                    b2 != Nat.0 and (forall(p: Nat) {
                        p.is_prime implies tail.count(p) <= count_prime_factor(p, b2)
                    }) implies product[Nat](tail).divides(b2)
                }
                tail_ih
                forall(b: Nat) {
                    if b != Nat.0 and (forall(p: Nat) {
                        p.is_prime implies List.cons(head, tail).count(p) <= count_prime_factor(p, b)
                    }) {
                        product_all_prime_divides_step(head, tail, b)
                        product[Nat](List.cons(head, tail)).divides(b)
                    }
                }
            }
            q(List.cons(head, tail))
        }
    }
    List.induction(function(l: List[Nat]) { q(l) })
    forall(l: List[Nat]) { q(l) }
    q(la)
}

/// Backward direction of the divisibility characterisation.
theorem count_prime_factor_le_imp_divides(a: Nat, b: Nat) {
    a != Nat.0 and b != Nat.0 and (forall(p: Nat) {
        p.is_prime implies count_prime_factor(p, a) <= count_prime_factor(p, b)
    }) implies a.divides(b)
} by {
    if a != Nat.0 and b != Nat.0 and (forall(p: Nat) {
        p.is_prime implies count_prime_factor(p, a) <= count_prime_factor(p, b)
    }) {
        Nat.1 <= a
        prime_factorisation_all_prime(a)
        prime_factorisation_product(a)
        all_prime(prime_factorisation(a))
        product[Nat](prime_factorisation(a)) = a
        forall(p: Nat) {
            if p.is_prime {
                count_prime_factor(p, a) <= count_prime_factor(p, b)
                prime_factorisation(a).count(p) <= count_prime_factor(p, b)
            }
        }
        product_all_prime_divides(prime_factorisation(a))
        if b != Nat.0 and (forall(p: Nat) {
            p.is_prime implies prime_factorisation(a).count(p) <= count_prime_factor(p, b)
        }) {
            product[Nat](prime_factorisation(a)).divides(b)
        }
        product[Nat](prime_factorisation(a)).divides(b)
        a.divides(b)
    }
}

/// The full divisibility characterisation: nonzero `a` divides nonzero `b`
/// iff every prime count of `a` is bounded by the matching count of `b`.
theorem divides_iff_count_prime_factor_le(a: Nat, b: Nat) {
    a != Nat.0 and b != Nat.0 implies (
        a.divides(b) = forall(p: Nat) {
            p.is_prime implies count_prime_factor(p, a) <= count_prime_factor(p, b)
        }
    )
} by {
    if a != Nat.0 and b != Nat.0 {
        if a.divides(b) {
            forall(p: Nat) {
                divides_imp_count_prime_factor_le(p, a, b)
            }
        }
        if forall(p: Nat) {
            p.is_prime implies count_prime_factor(p, a) <= count_prime_factor(p, b)
        } {
            count_prime_factor_le_imp_divides(a, b)
        }
    }
}

/// The prime-count of a prime power, by induction on the exponent.
theorem count_prime_factor_pow(p: Nat, k: Nat) {
    p.is_prime implies count_prime_factor(p, p.pow(k)) = k
} by {
    if p.is_prime {
        Nat.1 < p
        p != Nat.0
        let f: Nat -> Bool = function(x: Nat) {
            count_prime_factor(p, p.pow(x)) = x
        }
        p.pow(Nat.0) = Nat.1
        count_prime_factor_one(p)
        f(Nat.0)
        forall(x: Nat) {
            if f(x) {
                exp_ne_zero(p, x)
                p.pow(x) != Nat.0
                p.pow(x.suc) = p * p.pow(x)
                count_prime_factor_mul(p, p, p.pow(x))
                count_prime_factor(p, p * p.pow(x)) =
                    count_prime_factor(p, p) + count_prime_factor(p, p.pow(x))
                count_prime_factor_self(p)
                count_prime_factor(p, p) = Nat.1
                count_prime_factor(p, p.pow(x)) = x
                count_prime_factor(p, p.pow(x.suc)) = Nat.1 + x
                Nat.1 + x = x.suc
                count_prime_factor(p, p.pow(x.suc)) = x.suc
                f(x.suc)
            }
        }
        f(k)
        count_prime_factor(p, p.pow(k)) = k
    }
}

/// At a different prime, the count of a prime power vanishes.
theorem count_prime_factor_pow_other(p: Nat, q: Nat, k: Nat) {
    p.is_prime and q.is_prime and p != q
        implies count_prime_factor(q, p.pow(k)) = Nat.0
} by {
    if p.is_prime and q.is_prime and p != q {
        Nat.1 < p
        p != Nat.0
        let f: Nat -> Bool = function(x: Nat) {
            count_prime_factor(q, p.pow(x)) = Nat.0
        }
        p.pow(Nat.0) = Nat.1
        count_prime_factor_one(q)
        f(Nat.0)
        forall(x: Nat) {
            if f(x) {
                exp_ne_zero(p, x)
                p.pow(x) != Nat.0
                p.pow(x.suc) = p * p.pow(x)
                count_prime_factor_mul(q, p, p.pow(x))
                count_prime_factor(q, p * p.pow(x)) =
                    count_prime_factor(q, p) + count_prime_factor(q, p.pow(x))
                count_prime_factor_other_prime(q, p)
                count_prime_factor(q, p) = Nat.0
                count_prime_factor(q, p.pow(x)) = Nat.0
                count_prime_factor(q, p.pow(x.suc)) = Nat.0 + Nat.0
                count_prime_factor(q, p.pow(x.suc)) = Nat.0
                f(x.suc)
            }
        }
        f(k)
        count_prime_factor(q, p.pow(k)) = Nat.0
    }
}

/// At any prime `q`, the count of `p^k` is bounded by the count in `n` whenever
/// `k <= count_prime_factor(p, n)`. Used by the backward direction of
/// `prime_pow_divides_iff`.
theorem prime_pow_count_le_at(p: Nat, k: Nat, n: Nat, q: Nat) {
    p.is_prime and q.is_prime and k <= count_prime_factor(p, n)
        implies count_prime_factor(q, p.pow(k)) <= count_prime_factor(q, n)
} by {
    if p.is_prime and q.is_prime and k <= count_prime_factor(p, n) {
        if q = p {
            count_prime_factor_pow(p, k)
            count_prime_factor(q, p.pow(k)) = k
            count_prime_factor(q, p.pow(k)) <= count_prime_factor(q, n)
        } else {
            count_prime_factor_pow_other(p, q, k)
            count_prime_factor(q, p.pow(k)) = Nat.0
            count_prime_factor(q, p.pow(k)) <= count_prime_factor(q, n)
        }
    }
}

/// Forward direction of `prime_pow_divides_iff`.
theorem prime_pow_divides_imp_count_le(p: Nat, k: Nat, n: Nat) {
    p.is_prime and n != Nat.0 and p.pow(k).divides(n)
        implies k <= count_prime_factor(p, n)
} by {
    if p.is_prime and n != Nat.0 and p.pow(k).divides(n) {
        Nat.1 < p
        p != Nat.0
        exp_ne_zero(p, k)
        p.pow(k) != Nat.0
        divides_imp_count_prime_factor_le(p, p.pow(k), n)
        count_prime_factor_pow(p, k)
        count_prime_factor(p, p.pow(k)) = k
        k <= count_prime_factor(p, n)
    }
}

/// Backward direction of `prime_pow_divides_iff`.
theorem count_le_imp_prime_pow_divides(p: Nat, k: Nat, n: Nat) {
    p.is_prime and n != Nat.0 and k <= count_prime_factor(p, n)
        implies p.pow(k).divides(n)
} by {
    if p.is_prime and n != Nat.0 and k <= count_prime_factor(p, n) {
        Nat.1 < p
        p != Nat.0
        exp_ne_zero(p, k)
        p.pow(k) != Nat.0
        forall(q: Nat) {
            if q.is_prime {
                prime_pow_count_le_at(p, k, n, q)
                count_prime_factor(q, p.pow(k)) <= count_prime_factor(q, n)
            }
        }
        count_prime_factor_le_imp_divides(p.pow(k), n)
        p.pow(k).divides(n)
    }
}

/// `p^k` divides nonzero `n` iff `k <= count_prime_factor(p, n)`.
theorem prime_pow_divides_iff(p: Nat, k: Nat, n: Nat) {
    p.is_prime and n != Nat.0 implies (
        p.pow(k).divides(n) = (k <= count_prime_factor(p, n))
    )
} by {
    if p.is_prime and n != Nat.0 {
        if p.pow(k).divides(n) {
            prime_pow_divides_imp_count_le(p, k, n)
            k <= count_prime_factor(p, n)
        }
        if k <= count_prime_factor(p, n) {
            count_le_imp_prime_pow_divides(p, k, n)
            p.pow(k).divides(n)
        }
        p.pow(k).divides(n) = (k <= count_prime_factor(p, n))
    }
}

/// The prime count of a gcd is the pointwise minimum of the prime counts.
theorem count_prime_factor_gcd(p: Nat, a: Nat, b: Nat) {
    a != Nat.0 and b != Nat.0 implies
        count_prime_factor(p, a.gcd(b)) =
            count_prime_factor(p, a).min(count_prime_factor(p, b))
} by {
    if a != Nat.0 and b != Nat.0 {
        gcd_nonzero_left(a, b)
        a.gcd(b) != Nat.0
        gcd_divides_left(a, b)
        gcd_divides_right(a, b)
        if not p.is_prime {
            count_prime_factor_non_prime(p, a)
            count_prime_factor_non_prime(p, b)
            count_prime_factor_non_prime(p, a.gcd(b))
            count_prime_factor(p, a) = Nat.0
            count_prime_factor(p, b) = Nat.0
            count_prime_factor(p, a.gcd(b)) = Nat.0
            Nat.0.min(Nat.0) = Nat.0
            count_prime_factor(p, a).min(count_prime_factor(p, b)) = Nat.0
            count_prime_factor(p, a.gcd(b)) =
                count_prime_factor(p, a).min(count_prime_factor(p, b))
        }
        if p.is_prime {
            divides_imp_count_prime_factor_le(p, a.gcd(b), a)
            divides_imp_count_prime_factor_le(p, a.gcd(b), b)
            let m: Nat = count_prime_factor(p, a).min(count_prime_factor(p, b))
            m <= count_prime_factor(p, a)
            m <= count_prime_factor(p, b)
            count_le_imp_prime_pow_divides(p, m, a)
            count_le_imp_prime_pow_divides(p, m, b)
            divides_gcd(p.pow(m), a, b)
            p.pow(m).divides(a.gcd(b))
            prime_pow_divides_imp_count_le(p, m, a.gcd(b))
            m <= count_prime_factor(p, a.gcd(b))
            count_prime_factor(p, a.gcd(b)) <= m
            count_prime_factor(p, a.gcd(b)) = m
            count_prime_factor(p, a.gcd(b)) =
                count_prime_factor(p, a).min(count_prime_factor(p, b))
        }
        count_prime_factor(p, a.gcd(b)) =
            count_prime_factor(p, a).min(count_prime_factor(p, b))
    }
}

/// `min(a, b) + max(a, b) = a + b` on `Nat`, via case-split on the linear order.
theorem nat_min_plus_max(a: Nat, b: Nat) {
    a.min(b) + a.max(b) = a + b
} by {
    if a <= b {
        a.min(b) = a
        a.max(b) = b
        a.min(b) + a.max(b) = a + b
    } else {
        b <= a
        a.min(b) = b
        a.max(b) = a
        b + a = a + b
        a.min(b) + a.max(b) = a + b
    }
}

/// `a.lcm(b) != Nat.0` whenever both `a` and `b` are nonzero, derived from
/// `gcd_mul_lcm`.
theorem lcm_nonzero(a: Nat, b: Nat) {
    a != Nat.0 and b != Nat.0 implies a.lcm(b) != Nat.0
} by {
    if a != Nat.0 and b != Nat.0 {
        gcd_mul_lcm(a, b)
        a.gcd(b) * a.lcm(b) = a * b
        if a.lcm(b) = Nat.0 {
            a.gcd(b) * Nat.0 = Nat.0
            a * b = Nat.0
            mul_to_zero(a, b)
            false
        }
        a.lcm(b) != Nat.0
    }
}

/// Helper for `count_prime_factor_lcm`: the prime counts of `gcd` and `lcm`
/// sum to the prime counts of `a` and `b`.
theorem count_prime_factor_gcd_plus_lcm(p: Nat, a: Nat, b: Nat) {
    a != Nat.0 and b != Nat.0 implies
        count_prime_factor(p, a.gcd(b)) + count_prime_factor(p, a.lcm(b)) =
            count_prime_factor(p, a) + count_prime_factor(p, b)
} by {
    if a != Nat.0 and b != Nat.0 {
        gcd_nonzero_left(a, b)
        a.gcd(b) != Nat.0
        lcm_nonzero(a, b)
        a.lcm(b) != Nat.0
        gcd_mul_lcm(a, b)
        let lhs: Nat = a.gcd(b) * a.lcm(b)
        let rhs: Nat = a * b
        lhs = rhs
        count_prime_factor(p, lhs) = count_prime_factor(p, rhs)
        count_prime_factor_mul(p, a.gcd(b), a.lcm(b))
        count_prime_factor(p, lhs) =
            count_prime_factor(p, a.gcd(b)) + count_prime_factor(p, a.lcm(b))
        count_prime_factor_mul(p, a, b)
        count_prime_factor(p, rhs) =
            count_prime_factor(p, a) + count_prime_factor(p, b)
        count_prime_factor(p, a.gcd(b)) + count_prime_factor(p, a.lcm(b)) =
            count_prime_factor(p, a) + count_prime_factor(p, b)
    }
}

/// The prime count of an lcm is the pointwise maximum of the prime counts,
/// derived from the gcd characterisation via `gcd * lcm = a * b`.
theorem count_prime_factor_lcm(p: Nat, a: Nat, b: Nat) {
    a != Nat.0 and b != Nat.0 implies
        count_prime_factor(p, a.lcm(b)) =
            count_prime_factor(p, a).max(count_prime_factor(p, b))
} by {
    if a != Nat.0 and b != Nat.0 {
        count_prime_factor_gcd_plus_lcm(p, a, b)
        count_prime_factor(p, a.gcd(b)) + count_prime_factor(p, a.lcm(b)) = count_prime_factor(p, a) + count_prime_factor(p, b)
        count_prime_factor_gcd(p, a, b)
        count_prime_factor(p, a.gcd(b)) = count_prime_factor(p, a).min(count_prime_factor(p, b))
        count_prime_factor(p, a).min(count_prime_factor(p, b)) + count_prime_factor(p, a.lcm(b)) = count_prime_factor(p, a) + count_prime_factor(p, b)
        nat_min_plus_max(count_prime_factor(p, a), count_prime_factor(p, b))
        count_prime_factor(p, a).min(count_prime_factor(p, b)) + count_prime_factor(p, a).max(count_prime_factor(p, b)) = count_prime_factor(p, a) + count_prime_factor(p, b)
        count_prime_factor(p, a).min(count_prime_factor(p, b)) + count_prime_factor(p, a.lcm(b)) = count_prime_factor(p, a).min(count_prime_factor(p, b)) + count_prime_factor(p, a).max(count_prime_factor(p, b))
        add_cancels_left(count_prime_factor(p, a).min(count_prime_factor(p, b)), count_prime_factor(p, a.lcm(b)), count_prime_factor(p, a).max(count_prime_factor(p, b)))
        count_prime_factor(p, a.lcm(b)) = count_prime_factor(p, a).max(count_prime_factor(p, b))
    }
}

/// True for primes whose multiplicity in `a` is strictly larger than in `b`.
define lcm_left_prime_pred(a: Nat, b: Nat) -> (Nat -> Bool) {
    function(p: Nat) {
        count_prime_factor(p, b) < count_prime_factor(p, a)
    }
}

/// True for primes whose multiplicity in `b` is at least that in `a`.
define lcm_right_prime_pred(a: Nat, b: Nat) -> (Nat -> Bool) {
    function(p: Nat) {
        count_prime_factor(p, a) <= count_prime_factor(p, b)
    }
}

/// The factor of `a` carrying prime powers whose multiplicity dominates `b`.
define lcm_left_coprime_factor(a: Nat, b: Nat) -> Nat {
    selected_prime_factor_product(a, lcm_left_prime_pred(a, b))
}

/// The factor of `b` carrying all remaining maximal prime powers.
define lcm_right_coprime_factor(a: Nat, b: Nat) -> Nat {
    selected_prime_factor_product(b, lcm_right_prime_pred(a, b))
}

/// Two positive naturals have coprime divisors whose product is their least
/// common multiple.
theorem coprime_factors_mul_lcm(a: Nat, b: Nat) {
    a != Nat.0 and b != Nat.0 implies exists(left: Nat, right: Nat) {
        left != Nat.0 and right != Nat.0
            and left.divides(a) and right.divides(b)
            and left.coprime(right) and left * right = a.lcm(b)
    }
} by {
    if a != Nat.0 and b != Nat.0 {
        let lp = lcm_left_prime_pred(a, b)
        let rp = lcm_right_prime_pred(a, b)
        let left = lcm_left_coprime_factor(a, b)
        let right = lcm_right_coprime_factor(a, b)
        left = selected_prime_factor_product(a, lp)
        right = selected_prime_factor_product(b, rp)
        selected_prime_factor_product_nonzero(a, lp)
        selected_prime_factor_product_nonzero(b, rp)
        left != Nat.0
        right != Nat.0

        forall(p: Nat) {
            if p.is_prime {
                count_prime_factor_selected_product(p, a, lp)
                if lp(p) {
                    count_prime_factor(p, left) = count_prime_factor(p, a)
                    lte_ref(count_prime_factor(p, a))
                    count_prime_factor(p, left) <= count_prime_factor(p, a)
                } else {
                    count_prime_factor(p, left) = Nat.0
                    Nat.0 <= count_prime_factor(p, a)
                    count_prime_factor(p, left) <= count_prime_factor(p, a)
                }
                count_prime_factor(p, left) <= count_prime_factor(p, a)
            }
        }
        forall(p: Nat) {
            p.is_prime implies count_prime_factor(p, left) <= count_prime_factor(p, a)
        }
        count_prime_factor_le_imp_divides(left, a)
        left.divides(a)

        forall(p: Nat) {
            if p.is_prime {
                count_prime_factor_selected_product(p, b, rp)
                if rp(p) {
                    count_prime_factor(p, right) = count_prime_factor(p, b)
                    lte_ref(count_prime_factor(p, b))
                    count_prime_factor(p, right) <= count_prime_factor(p, b)
                } else {
                    count_prime_factor(p, right) = Nat.0
                    Nat.0 <= count_prime_factor(p, b)
                    count_prime_factor(p, right) <= count_prime_factor(p, b)
                }
                count_prime_factor(p, right) <= count_prime_factor(p, b)
            }
        }
        forall(p: Nat) {
            p.is_prime implies count_prime_factor(p, right) <= count_prime_factor(p, b)
        }
        count_prime_factor_le_imp_divides(right, b)
        right.divides(b)

        forall(p: Nat) {
            if p.is_prime {
                if p.divides(left) and p.divides(right) {
                    count_prime_factor_self(p)
                    count_prime_factor(p, p) = Nat.1
                    divides_imp_count_prime_factor_le(p, p, left)
                    divides_imp_count_prime_factor_le(p, p, right)
                    Nat.1 <= count_prime_factor(p, left)
                    Nat.1 <= count_prime_factor(p, right)
                    if count_prime_factor(p, b) < count_prime_factor(p, a) {
                        lp(p)
                        not rp(p)
                        count_prime_factor_selected_product(p, b, rp)
                        count_prime_factor(p, right) = Nat.0
                        Nat.1 <= Nat.0
                        false
                    } else {
                        not lp(p)
                        count_prime_factor_selected_product(p, a, lp)
                        count_prime_factor(p, left) = Nat.0
                        Nat.1 <= Nat.0
                        false
                    }
                }
                not (p.divides(left) and p.divides(right))
            }
        }
        coprime_iff_no_shared_prime_factor(left, right)
        left.coprime(right)

        lcm_nonzero(a, b)
        a.lcm(b) != Nat.0
        left * right != Nat.0
        forall(p: Nat) {
            if p.is_prime {
                count_prime_factor_mul(p, left, right)
                count_prime_factor_selected_product(p, a, lp)
                count_prime_factor_selected_product(p, b, rp)
                count_prime_factor_lcm(p, a, b)
                count_prime_factor(p, left * right) =
                    count_prime_factor(p, left) + count_prime_factor(p, right)
                count_prime_factor(p, a.lcm(b)) =
                    count_prime_factor(p, a).max(count_prime_factor(p, b))
                if count_prime_factor(p, b) < count_prime_factor(p, a) {
                    lp(p)
                    not rp(p)
                    count_prime_factor(p, left) = count_prime_factor(p, a)
                    count_prime_factor(p, right) = Nat.0
                    count_prime_factor(p, a).max(count_prime_factor(p, b)) =
                        count_prime_factor(p, a)
                    count_prime_factor(p, left * right) =
                        count_prime_factor(p, a) + Nat.0
                    count_prime_factor(p, a) + Nat.0 = count_prime_factor(p, a)
                    count_prime_factor(p, a.lcm(b)) = count_prime_factor(p, a)
                    count_prime_factor(p, left * right) =
                        count_prime_factor(p, a.lcm(b))
                } else {
                    not lp(p)
                    rp(p)
                    count_prime_factor(p, left) = Nat.0
                    count_prime_factor(p, right) = count_prime_factor(p, b)
                    count_prime_factor(p, a).max(count_prime_factor(p, b)) =
                        count_prime_factor(p, b)
                    count_prime_factor(p, left * right) =
                        Nat.0 + count_prime_factor(p, b)
                    Nat.0 + count_prime_factor(p, b) = count_prime_factor(p, b)
                    count_prime_factor(p, a.lcm(b)) = count_prime_factor(p, b)
                    count_prime_factor(p, left * right) =
                        count_prime_factor(p, a.lcm(b))
                }
                count_prime_factor(p, left * right) =
                    count_prime_factor(p, a.lcm(b))
            }
        }
        count_prime_factor_ext(left * right, a.lcm(b))
        left * right = a.lcm(b)
        exists(left0: Nat, right0: Nat) {
            left0 != Nat.0 and right0 != Nat.0
                and left0.divides(a) and right0.divides(b)
                and left0.coprime(right0) and left0 * right0 = a.lcm(b)
        }
    }
}
