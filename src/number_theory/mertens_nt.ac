// The Mertens function: `M(x) = sum_{n <= x} mu(n)` and its small values.
//
// This file restates the Mertens function from `mobius_inversion.ac` in the range-sum form
// used by the rest of the analytic number theory layer, records the small values, and
// comments the Möbius-inversion connection back to the prime counting function.  It is a
// separate file from `analytic_nt.ac` because the module graph of `mobius_inversion.ac`
// (through `fermat.ac` and `congruence.ac`) and the module graph of `chebyshev.ac` (through
// `interface.ac`) each define the `Nat.congr_mod` attribute, so no single module can import
// both.

from nat import Nat
from int import Int
from list import List, map, sum
from data.nat.nat_range_sum import range_sum
from data.nat.nat_range_sum_bridge import range_sum_eq_list_sum
from number_theory.mobius_inversion import mertens, nat_mobius, nat_mobius_zero,
    nat_mobius_one, nat_mobius_prime, mertens_zero, mertens_one
from number_theory.falling_product import nat_two_prime
numerals Nat
numerals Int

/// The Mertens function `M(x) = sum_{n <= x} mu(n)`.
///
/// The definition in `mobius_inversion.ac` sums the Möbius function over the range `0..x`;
/// the term at zero is `mu(0) = 0`, so the value is the classical Mertens function.
/// Restated here as a range sum so that the analytic number theory layer stands alone.
theorem mertens_eq_range_sum_mobius(x: Nat) {
    mertens(x) = range_sum(nat_mobius, x.suc)
} by {
    mertens(x) = sum(map(x.suc.range, nat_mobius))
    range_sum_eq_list_sum(nat_mobius, x.suc)
    range_sum(nat_mobius, x.suc) = sum(map(x.suc.range, nat_mobius))
    mertens(x) = range_sum(nat_mobius, x.suc)
}

/// `M(0) = mu(0) = 0`.
theorem mertens_zero_restated {
    mertens(Nat.0) = Int.0
} by {
    mertens_zero
    mertens(Nat.0) = Int.0
}

/// `M(1) = mu(0) + mu(1) = 1`.
theorem mertens_one_restated {
    mertens(Nat.1) = Int.1
} by {
    mertens_one
    mertens(Nat.1) = Int.1
}

/// `M(2) = mu(0) + mu(1) + mu(2) = 0 + 1 - 1 = 0`.
theorem mertens_two {
    mertens(Nat.2) = Int.0
} by {
    mertens(Nat.2) = sum(map(Nat.2.suc.range, nat_mobius))
    Nat.2.suc = Nat.3
    Nat.0.range = List.nil[Nat]
    Nat.1.range = Nat.0.range.append(Nat.0)
    Nat.1.range = List.cons(Nat.0, List.nil[Nat])
    Nat.2.range = Nat.1.range.append(Nat.1)
    List.cons(Nat.0, List.nil[Nat]).append(Nat.1) =
        List.cons(Nat.0, List.cons(Nat.1, List.nil[Nat]))
    Nat.2.range = List.cons(Nat.0, List.cons(Nat.1, List.nil[Nat]))
    Nat.3.range = Nat.2.range.append(Nat.2)
    List.cons(Nat.1, List.nil[Nat]).append(Nat.2) =
        List.cons(Nat.1, List.cons(Nat.2, List.nil[Nat]))
    List.cons(Nat.0, List.cons(Nat.1, List.nil[Nat])).append(Nat.2) =
        List.cons(Nat.0, List.cons(Nat.1, List.cons(Nat.2, List.nil[Nat])))
    Nat.3.range = List.cons(Nat.0, List.cons(Nat.1, List.cons(Nat.2, List.nil[Nat])))
    map(List.cons(Nat.0, List.cons(Nat.1, List.cons(Nat.2, List.nil[Nat]))), nat_mobius) =
        List.cons(nat_mobius(Nat.0),
            map(List.cons(Nat.1, List.cons(Nat.2, List.nil[Nat])), nat_mobius))
    map(List.cons(Nat.1, List.cons(Nat.2, List.nil[Nat])), nat_mobius) =
        List.cons(nat_mobius(Nat.1), map(List.cons(Nat.2, List.nil[Nat]), nat_mobius))
    map(List.cons(Nat.2, List.nil[Nat]), nat_mobius) =
        List.cons(nat_mobius(Nat.2), map(List.nil[Nat], nat_mobius))
    map(List.nil[Nat], nat_mobius) = List.nil[Int]
    sum(List.cons(nat_mobius(Nat.0),
        List.cons(nat_mobius(Nat.1),
            List.cons(nat_mobius(Nat.2), List.nil[Int])))) =
        nat_mobius(Nat.0) +
            sum(List.cons(nat_mobius(Nat.1), List.cons(nat_mobius(Nat.2), List.nil[Int])))
    sum(List.cons(nat_mobius(Nat.1), List.cons(nat_mobius(Nat.2), List.nil[Int]))) =
        nat_mobius(Nat.1) + sum(List.cons(nat_mobius(Nat.2), List.nil[Int]))
    sum(List.cons(nat_mobius(Nat.2), List.nil[Int])) =
        nat_mobius(Nat.2) + sum(List.nil[Int])
    sum(List.nil[Int]) = Int.0
    nat_mobius_zero
    nat_mobius(Nat.0) = Int.0
    nat_mobius_one
    nat_mobius(Nat.1) = Int.1
    nat_two_prime
    Nat.2.is_prime
    nat_mobius_prime(Nat.2)
    nat_mobius(Nat.2) = -Int.1
    Int.0 + Int.1 + -Int.1 + Int.0 = Int.0
    sum(map(Nat.3.range, nat_mobius)) = Int.0
    mertens(Nat.2) = Int.0
}

// The Möbius-inversion connection from `M` back to the primes: the fundamental identity
// `sum_{d | n} mu(d) = 1` if `n = 1` and `0` otherwise (proved in `mobius_inversion.ac` as
// `nat_mobius_divisor_sum`) is the engine of Möbius inversion, which expresses any function
// `g(n) = sum_{d | n} f(d)` as `f(n) = sum_{d | n} mu(d) * g(n / d)`.  Inversion applied to
// the identity function `n = sum_{d | n} phi(d)` recovers the totient from the divisor sums,
// and applied to the divisor-counting function it recovers the indicator of the primes;
// the general inversion theorem itself is recorded, unproved, in `mobius_inversion.ac`.
