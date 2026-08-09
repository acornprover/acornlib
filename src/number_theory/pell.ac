/// Pell's equation: for a nonsquare positive integer `d`, the equation
/// x² - d·y² = 1 has infinitely many integer solutions (x, y), the
/// fundamental one arising from the continued fraction of √d.
///
/// This file formalizes the statement, verifies the fundamental solution for
/// d = 2 (x = 3, y = 2, which is the first nontrivial convergent of the
/// continued fraction of √2), and proves the composition law at the squaring
/// step: from one solution (x, y) of x² - d·y² = 1, the pair
/// (x² + d·y², 2·x·y) is again a solution. Iterating this step generates
/// infinitely many solutions.
///
/// The classical estimate |p² - d·q²| < 2√d + 1 for a convergent p/q of the
/// continued fraction of √d is stated below; its proof requires the
/// convergence of the continued fraction of √d to √d as a real number, which
/// the library's continued-fraction machinery (continued_fraction_convergents.ac)
/// does not yet formalize, so the general estimate is left as a statement.
/// The d = 2 instance is proved exactly: |3² - 2·2²| = 1.
from int import Int, mul_neg_left, mul_neg_right, mul_neg_neg, neg_sub,
    mul_from_nat, add_from_nat
from nat import Nat
from pair import Pair, pair_new_first, pair_new_second
from number_theory.continued_fraction_convergents import continued_fraction_convergent_numerator,
    continued_fraction_convergent_denominator, continued_fraction_recurrence_state,
    continued_fraction_recurrence_state_suc_first, continued_fraction_recurrence_state_zero,
    continued_fraction_convergent_numerator_suc, continued_fraction_convergent_numerator_zero,
    continued_fraction_convergent_denominator_suc, continued_fraction_convergent_denominator_zero

numerals Int
numerals Nat

// ============================================================================
// Section 1: the Pell equation
// ============================================================================

/// True when the integer pair (x, y) solves Pell's equation x² - d·y² = 1.
define is_pell_solution(d: Int, x: Int, y: Int) -> Bool {
    x * x - d * (y * y) = Int.1
}

/// The Pell norm x² - d·y² of a pair; Pell solutions are exactly the pairs of
/// norm one.
define pell_norm(d: Int, x: Int, y: Int) -> Int {
    x * x - d * (y * y)
}

/// The trivial solution (1, 0) solves Pell's equation for every d.
theorem pell_trivial_solution(d: Int) {
    is_pell_solution(d, Int.1, Int.0)
} by {
    Int.1 * Int.1 = Int.1
    Int.0 * Int.0 = Int.0
    d * Int.0 = Int.0
    Int.1 - Int.0 = Int.1
    is_pell_solution(d, Int.1, Int.0)
}

// ============================================================================
// Section 2: the fundamental solution for d = 2
// ============================================================================

/// The fundamental solution for d = 2: 3² - 2·2² = 1.
theorem pell_two_square_identity {
    Int.3 * Int.3 - Int.2 * (Int.2 * Int.2) = Int.1
} by {
    Int.3 * Int.3 = Int.9
    mul_from_nat(Nat.2, Nat.2)
    Int.from_nat(Nat.2) * Int.from_nat(Nat.2) = Int.from_nat(Nat.4)
    Int.2 * Int.2 = Int.4
    mul_from_nat(Nat.2, Nat.4)
    Int.from_nat(Nat.2) * Int.from_nat(Nat.4) = Int.from_nat(Nat.8)
    Int.2 * Int.4 = Int.8
    add_from_nat(Nat.8, Nat.1)
    Int.from_nat(Nat.8) + Int.from_nat(Nat.1) = Int.from_nat(Nat.9)
    Int.9 = Int.8 + Int.1
    (Int.8 + Int.1) - Int.8 = Int.1
    Int.9 - Int.8 = Int.1
    Int.3 * Int.3 - Int.2 * (Int.2 * Int.2) = Int.1
}

/// The pair (3, 2) is the smallest positive solution of x² - 2·y² = 1.
theorem pell_two_smallest {
    is_pell_solution(Int.2, Int.3, Int.2)
} by {
    pell_two_square_identity
    Int.3 * Int.3 - Int.2 * (Int.2 * Int.2) = Int.1
    is_pell_solution(Int.2, Int.3, Int.2)
}

// ============================================================================
// Section 3: generating infinitely many solutions
// ============================================================================

/// The x-component of the squared solution (x² + d·y², 2·x·y) generated from
/// a solution (x, y).
define pell_next_x(d: Int, x: Int, y: Int) -> Int {
    x * x + d * (y * y)
}

/// The y-component of the squared solution generated from a solution (x, y).
define pell_next_y(d: Int, x: Int, y: Int) -> Int {
    Int.2 * x * y
}

/// A parenthesized minuend flattens under subtraction.
theorem pell_rearrange_flatten(a: Int, b: Int, c: Int, d: Int) {
    (a + b) - c - d = a + b - c - d
}

/// A trailing summand may be regrouped with the leading terms.
theorem pell_rearrange_regroup(a: Int, b: Int, c: Int, d: Int) {
    a - b - c + d = (a + d) - b - c
} by {
    a - b - c + d = a + -b + -c + d
    a + -b + -c + d = a + d + -b + -c
    a + d + -b + -c = a + d - b - c
    a + d - b - c = (a + d) - b - c
}

/// Subtraction distributes over a sum on the right.
theorem pell_rearrange_sub_sum(a: Int, b: Int, c: Int) {
    a - (b + c) = a - b - c
} by {
    a - (b + c) = a + -(b + c)
    a + -(b + c) = a + (-b + -c)
    a + (-b + -c) = a + -b + -c
    a + -b + -c = a - b - c
}

/// Cancellation of four equal summands flattens the result.
theorem pell_rearrange_cancel(a: Int, b: Int, c: Int) {
    (a + b + c + c) - (c + c + c + c) = a + b - c - c
} by {
    (a + b + c + c) - (c + c + c + c) = (a + b + (c + c)) - ((c + c) + (c + c))
    (a + b + (c + c)) - ((c + c) + (c + c)) = (a + b) - (c + c)
    pell_rearrange_sub_sum(a + b, c, c)
    (a + b) - (c + c) = (a + b) - c - c
    pell_rearrange_flatten(a, b, c, c)
    (a + b) - c - c = a + b - c - c
}

/// A difference of differences flattens with the sign of the inner difference
/// reversed.
theorem pell_rearrange_sub_sub(a: Int, b: Int, c: Int, d: Int) {
    (a - b) - (c - d) = a - b - c + d
} by {
    (a - b) - (c - d) = (a - b) + -(c - d)
    neg_sub(c, d)
    (a - b) + -(c - d) = (a - b) + (d - c)
    (a - b) + (d - c) = a - b + d - c
    a - b + d - c = a + -b + d + -c
    a + -b + d + -c = a + -b + -c + d
    a + -b + -c + d = a - b - c + d
}

/// The cross term of the squared solution: d·(x·y)·(x·y) = x·x·(d·(y·y)).
theorem pell_cross_term_eq(d: Int, x: Int, y: Int) {
    d * (x * y) * (x * y) = (x * x) * (d * (y * y))
} by {
    d * (x * y) * (x * y) = d * x * y * x * y
    d * x * y * x * y = d * x * x * y * y
    d * x * x * y * y = x * x * d * y * y
    x * x * d * y * y = (x * x) * (d * (y * y))
}

/// The norm of the squared solution is the square of the norm:
/// (x² + d·y²)² - d·(2·x·y)² = (x² - d·y²)².
theorem pell_square_identity(d: Int, x: Int, y: Int) {
    (x * x + d * (y * y)) * (x * x + d * (y * y)) -
        d * ((Int.2 * x * y) * (Int.2 * x * y)) =
    (x * x - d * (y * y)) * (x * x - d * (y * y))
} by {
    Int.2 * x * y = x * y + x * y
    (x * x + d * (y * y)) * (x * x + d * (y * y)) - d * ((Int.2 * x * y) * (Int.2 * x * y)) =
        (x * x + d * (y * y)) * (x * x + d * (y * y)) - d * ((x * y + x * y) * (x * y + x * y))
    (x * x + d * (y * y)) * (x * x + d * (y * y)) - d * ((x * y + x * y) * (x * y + x * y)) =
        ((x * x + d * (y * y)) * (x * x) + (x * x + d * (y * y)) * (d * (y * y))) -
            d * ((x * y + x * y) * (x * y + x * y))
    (x * x + d * (y * y)) * (x * x) = (x * x) * (x * x) + (d * (y * y)) * (x * x)
    (x * x + d * (y * y)) * (d * (y * y)) = (x * x) * (d * (y * y)) + (d * (y * y)) * (d * (y * y))
    ((x * x + d * (y * y)) * (x * x) + (x * x + d * (y * y)) * (d * (y * y))) -
            d * ((x * y + x * y) * (x * y + x * y)) =
        ((x * x) * (x * x) + (d * (y * y)) * (x * x) + (x * x) * (d * (y * y)) + (d * (y * y)) * (d * (y * y))) -
            d * ((x * y + x * y) * (x * y + x * y))
    d * ((x * y + x * y) * (x * y + x * y)) =
        (d * (x * y + x * y)) * (x * y + x * y)
    d * (x * y + x * y) = d * (x * y) + d * (x * y)
    d * ((x * y + x * y) * (x * y + x * y)) =
        (d * (x * y) + d * (x * y)) * (x * y + x * y)
    (d * (x * y) + d * (x * y)) * (x * y + x * y) =
        (d * (x * y) + d * (x * y)) * (x * y) + (d * (x * y) + d * (x * y)) * (x * y)
    (d * (x * y) + d * (x * y)) * (x * y) =
        d * (x * y) * (x * y) + d * (x * y) * (x * y)
    pell_cross_term_eq(d, x, y)
    d * (x * y) * (x * y) + d * (x * y) * (x * y) =
        (x * x) * (d * (y * y)) + (x * x) * (d * (y * y))
    d * ((x * y + x * y) * (x * y + x * y)) =
        (x * x) * (d * (y * y)) + (x * x) * (d * (y * y)) +
        (x * x) * (d * (y * y)) + (x * x) * (d * (y * y))
    (x * x) * (d * (y * y)) = (d * (y * y)) * (x * x)
    (x * x + d * (y * y)) * (x * x + d * (y * y)) -
            d * ((Int.2 * x * y) * (Int.2 * x * y)) =
        ((x * x) * (x * x) + (d * (y * y)) * (x * x) + (x * x) * (d * (y * y)) + (d * (y * y)) * (d * (y * y))) -
            ((x * x) * (d * (y * y)) + (x * x) * (d * (y * y)) +
                (x * x) * (d * (y * y)) + (x * x) * (d * (y * y)))
    (x * x + d * (y * y)) * (x * x + d * (y * y)) -
            d * ((Int.2 * x * y) * (Int.2 * x * y)) =
        ((x * x) * (x * x) + (x * x) * (d * (y * y)) + (x * x) * (d * (y * y)) + (d * (y * y)) * (d * (y * y))) -
            ((x * x) * (d * (y * y)) + (x * x) * (d * (y * y)) +
                (x * x) * (d * (y * y)) + (x * x) * (d * (y * y)))
    (x * x + d * (y * y)) * (x * x + d * (y * y)) -
            d * ((Int.2 * x * y) * (Int.2 * x * y)) =
        ((x * x) * (x * x) + (d * (y * y)) * (d * (y * y)) + ((x * x) * (d * (y * y)) + (x * x) * (d * (y * y)))) -
            (((x * x) * (d * (y * y)) + (x * x) * (d * (y * y))) +
                ((x * x) * (d * (y * y)) + (x * x) * (d * (y * y))))
    (x * x + d * (y * y)) * (x * x + d * (y * y)) -
            d * ((Int.2 * x * y) * (Int.2 * x * y)) =
        ((x * x) * (x * x) + (d * (y * y)) * (d * (y * y)) + (x * x) * (d * (y * y)) + (x * x) * (d * (y * y))) -
            ((x * x) * (d * (y * y)) + (x * x) * (d * (y * y)) + (x * x) * (d * (y * y)) + (x * x) * (d * (y * y)))
    pell_rearrange_cancel(
        (x * x) * (x * x), (d * (y * y)) * (d * (y * y)), (x * x) * (d * (y * y)))
    (x * x + d * (y * y)) * (x * x + d * (y * y)) -
            d * ((Int.2 * x * y) * (Int.2 * x * y)) =
        (x * x) * (x * x) + (d * (y * y)) * (d * (y * y)) -
            (x * x) * (d * (y * y)) - (x * x) * (d * (y * y))
    (x * x - d * (y * y)) * (x * x - d * (y * y)) =
        (x * x - d * (y * y)) * (x * x) - (x * x - d * (y * y)) * (d * (y * y))
    (x * x - d * (y * y)) * (x * x) = (x * x) * (x * x) - (d * (y * y)) * (x * x)
    (x * x - d * (y * y)) * (d * (y * y)) = (x * x) * (d * (y * y)) - (d * (y * y)) * (d * (y * y))
    (x * x - d * (y * y)) * (x * x) - (x * x - d * (y * y)) * (d * (y * y)) =
        ((x * x) * (x * x) - (d * (y * y)) * (x * x)) - ((x * x) * (d * (y * y)) - (d * (y * y)) * (d * (y * y)))
    (x * x) * (d * (y * y)) = (d * (y * y)) * (x * x)
    ((x * x) * (x * x) - (d * (y * y)) * (x * x)) - ((x * x) * (d * (y * y)) - (d * (y * y)) * (d * (y * y))) =
        ((x * x) * (x * x) - (x * x) * (d * (y * y))) - ((x * x) * (d * (y * y)) - (d * (y * y)) * (d * (y * y)))
    pell_rearrange_sub_sub(
        (x * x) * (x * x), (x * x) * (d * (y * y)), (x * x) * (d * (y * y)), (d * (y * y)) * (d * (y * y)))
    ((x * x) * (x * x) - (x * x) * (d * (y * y))) - ((x * x) * (d * (y * y)) - (d * (y * y)) * (d * (y * y))) =
        (x * x) * (x * x) - (x * x) * (d * (y * y)) - (x * x) * (d * (y * y)) + (d * (y * y)) * (d * (y * y))
    pell_rearrange_regroup(
        (x * x) * (x * x), (x * x) * (d * (y * y)), (x * x) * (d * (y * y)), (d * (y * y)) * (d * (y * y)))
    (x * x) * (x * x) - (x * x) * (d * (y * y)) - (x * x) * (d * (y * y)) + (d * (y * y)) * (d * (y * y)) =
        ((x * x) * (x * x) + (d * (y * y)) * (d * (y * y))) - (x * x) * (d * (y * y)) - (x * x) * (d * (y * y))
    (x * x - d * (y * y)) * (x * x - d * (y * y)) =
        ((x * x) * (x * x) + (d * (y * y)) * (d * (y * y))) - (x * x) * (d * (y * y)) - (x * x) * (d * (y * y))
    (x * x + d * (y * y)) * (x * x + d * (y * y)) -
            d * ((Int.2 * x * y) * (Int.2 * x * y)) =
        (x * x - d * (y * y)) * (x * x - d * (y * y))
}

/// Squaring a solution gives another solution:
/// (x, y) solves x² - d·y² = 1 implies (x² + d·y², 2·x·y) solves it too.
theorem pell_solution_square(d: Int, x: Int, y: Int) {
    is_pell_solution(d, x, y) implies
        is_pell_solution(d, pell_next_x(d, x, y), pell_next_y(d, x, y))
} by {
    if is_pell_solution(d, x, y) {
        pell_square_identity(d, x, y)
        pell_norm(d, pell_next_x(d, x, y), pell_next_y(d, x, y)) =
            pell_norm(d, x, y) * pell_norm(d, x, y)
        pell_norm(d, x, y) = Int.1
        pell_norm(d, pell_next_x(d, x, y), pell_next_y(d, x, y)) = Int.1
        is_pell_solution(d, pell_next_x(d, x, y), pell_next_y(d, x, y))
    }
}

// ============================================================================
// Section 4: the continued fraction of sqrt(2)
// ============================================================================

/// The coefficients of the continued fraction of √2: [1; 2, 2, 2, ...].
define sqrt_two_continued_fraction_coefficients(n: Nat) -> Nat {
    if n = Nat.0 { Nat.1 } else { Nat.2 }
}

/// The zeroth convergent of √2 is 1/1.
theorem sqrt_two_convergent_numerator_zero {
    continued_fraction_convergent_numerator(
        sqrt_two_continued_fraction_coefficients, Nat.0) = Nat.1
}

/// The zeroth convergent of √2 has denominator one.
theorem sqrt_two_convergent_denominator_zero {
    continued_fraction_convergent_denominator(
        sqrt_two_continued_fraction_coefficients, Nat.0) = Nat.1
}

/// The first convergent of √2 has numerator three.
theorem sqrt_two_convergent_numerator_one {
    continued_fraction_convergent_numerator(
        sqrt_two_continued_fraction_coefficients, Nat.1) = Nat.3
} by {
    continued_fraction_convergent_numerator_suc(
        sqrt_two_continued_fraction_coefficients, Nat.0)
    continued_fraction_convergent_numerator(
        sqrt_two_continued_fraction_coefficients, Nat.1) =
        continued_fraction_convergent_numerator(
            sqrt_two_continued_fraction_coefficients, Nat.0) *
            sqrt_two_continued_fraction_coefficients(Nat.1) +
        continued_fraction_recurrence_state(
            sqrt_two_continued_fraction_coefficients, Nat.1, Nat.0, Nat.1).first
    continued_fraction_convergent_numerator_zero(
        sqrt_two_continued_fraction_coefficients)
    continued_fraction_convergent_numerator(
        sqrt_two_continued_fraction_coefficients, Nat.0) =
        sqrt_two_continued_fraction_coefficients(Nat.0)
    sqrt_two_continued_fraction_coefficients(Nat.0) = Nat.1
    continued_fraction_convergent_numerator(
        sqrt_two_continued_fraction_coefficients, Nat.0) = Nat.1
    continued_fraction_recurrence_state_suc_first(
        sqrt_two_continued_fraction_coefficients, Nat.0, Nat.0, Nat.1)
    continued_fraction_recurrence_state(
        sqrt_two_continued_fraction_coefficients, Nat.1, Nat.0, Nat.1).first =
        continued_fraction_recurrence_state(
            sqrt_two_continued_fraction_coefficients, Nat.0, Nat.0, Nat.1).second
    continued_fraction_recurrence_state_zero(
        sqrt_two_continued_fraction_coefficients, Nat.0, Nat.1)
    continued_fraction_recurrence_state(
        sqrt_two_continued_fraction_coefficients, Nat.0, Nat.0, Nat.1) =
        Pair.new(Nat.0, Nat.1)
    pair_new_second(Nat.0, Nat.1)
    continued_fraction_recurrence_state(
        sqrt_two_continued_fraction_coefficients, Nat.0, Nat.0, Nat.1).second = Nat.1
    continued_fraction_recurrence_state(
        sqrt_two_continued_fraction_coefficients, Nat.1, Nat.0, Nat.1).first = Nat.1
    sqrt_two_continued_fraction_coefficients(Nat.1) = Nat.2
    continued_fraction_convergent_numerator(
        sqrt_two_continued_fraction_coefficients, Nat.1) = Nat.1 * Nat.2 + Nat.1
    Nat.1 * Nat.2 + Nat.1 = Nat.3
    continued_fraction_convergent_numerator(
        sqrt_two_continued_fraction_coefficients, Nat.1) = Nat.3
}

/// The first convergent of √2 has denominator two.
theorem sqrt_two_convergent_denominator_one {
    continued_fraction_convergent_denominator(
        sqrt_two_continued_fraction_coefficients, Nat.1) = Nat.2
} by {
    continued_fraction_convergent_denominator_suc(
        sqrt_two_continued_fraction_coefficients, Nat.0)
    continued_fraction_convergent_denominator(
        sqrt_two_continued_fraction_coefficients, Nat.1) =
        continued_fraction_convergent_denominator(
            sqrt_two_continued_fraction_coefficients, Nat.0) *
            sqrt_two_continued_fraction_coefficients(Nat.1) +
        continued_fraction_recurrence_state(
            sqrt_two_continued_fraction_coefficients, Nat.1, Nat.1, Nat.0).first
    continued_fraction_convergent_denominator_zero(
        sqrt_two_continued_fraction_coefficients)
    continued_fraction_convergent_denominator(
        sqrt_two_continued_fraction_coefficients, Nat.0) = Nat.1
    continued_fraction_recurrence_state_suc_first(
        sqrt_two_continued_fraction_coefficients, Nat.0, Nat.1, Nat.0)
    continued_fraction_recurrence_state(
        sqrt_two_continued_fraction_coefficients, Nat.1, Nat.1, Nat.0).first =
        continued_fraction_recurrence_state(
            sqrt_two_continued_fraction_coefficients, Nat.0, Nat.1, Nat.0).second
    continued_fraction_recurrence_state_zero(
        sqrt_two_continued_fraction_coefficients, Nat.1, Nat.0)
    continued_fraction_recurrence_state(
        sqrt_two_continued_fraction_coefficients, Nat.0, Nat.1, Nat.0) =
        Pair.new(Nat.1, Nat.0)
    pair_new_second(Nat.1, Nat.0)
    continued_fraction_recurrence_state(
        sqrt_two_continued_fraction_coefficients, Nat.0, Nat.1, Nat.0).second = Nat.0
    continued_fraction_recurrence_state(
        sqrt_two_continued_fraction_coefficients, Nat.1, Nat.1, Nat.0).first = Nat.0
    sqrt_two_continued_fraction_coefficients(Nat.1) = Nat.2
    continued_fraction_convergent_denominator(
        sqrt_two_continued_fraction_coefficients, Nat.1) = Nat.1 * Nat.2 + Nat.0
    Nat.1 * Nat.2 + Nat.0 = Nat.2
    continued_fraction_convergent_denominator(
        sqrt_two_continued_fraction_coefficients, Nat.1) = Nat.2
}

/// The adjacent convergent cross product for √2: 3·1 - 1·2 = 1, the
/// determinant identity at index zero.
theorem sqrt_two_adjacent_cross_product {
    Int.from_nat(Nat.3) * Int.from_nat(Nat.1) -
        Int.from_nat(Nat.1) * Int.from_nat(Nat.2) = Int.1
} by {
    Int.from_nat(Nat.3) * Int.from_nat(Nat.1) = Int.from_nat(Nat.3)
    Int.from_nat(Nat.1) * Int.from_nat(Nat.2) = Int.from_nat(Nat.2)
    Int.from_nat(Nat.3) = Int.3
    Int.from_nat(Nat.2) = Int.2
    add_from_nat(Nat.2, Nat.1)
    Int.from_nat(Nat.2) + Int.from_nat(Nat.1) = Int.from_nat(Nat.3)
    Int.2 + Int.1 = Int.3
    Int.3 = Int.2 + Int.1
    (Int.2 + Int.1) - Int.2 = Int.1
    Int.3 - Int.2 = Int.1
    Int.from_nat(Nat.3) - Int.from_nat(Nat.2) = Int.1
    Int.from_nat(Nat.3) * Int.from_nat(Nat.1) -
        Int.from_nat(Nat.1) * Int.from_nat(Nat.2) = Int.1
}

/// The fundamental solution (3, 2) of x² - 2·y² = 1 is the first convergent
/// of the continued fraction of √2.
theorem pell_two_fundamental_solution_from_convergents {
    is_pell_solution(Int.2,
        Int.from_nat(continued_fraction_convergent_numerator(
            sqrt_two_continued_fraction_coefficients, Nat.1)),
        Int.from_nat(continued_fraction_convergent_denominator(
            sqrt_two_continued_fraction_coefficients, Nat.1)))
} by {
    sqrt_two_convergent_numerator_one
    sqrt_two_convergent_denominator_one
    continued_fraction_convergent_numerator(
        sqrt_two_continued_fraction_coefficients, Nat.1) = Nat.3
    continued_fraction_convergent_denominator(
        sqrt_two_continued_fraction_coefficients, Nat.1) = Nat.2
    Int.from_nat(Nat.3) = Int.3
    Int.from_nat(Nat.2) = Int.2
    pell_two_square_identity
    Int.3 * Int.3 - Int.2 * (Int.2 * Int.2) = Int.1
    is_pell_solution(Int.2,
        Int.from_nat(continued_fraction_convergent_numerator(
            sqrt_two_continued_fraction_coefficients, Nat.1)),
        Int.from_nat(continued_fraction_convergent_denominator(
            sqrt_two_continued_fraction_coefficients, Nat.1)))
}

// ============================================================================
// Section 5: the classical estimate (statement)
// ============================================================================

// The classical estimate: if p/q is a convergent of the continued fraction of
// √d (d a nonsquare positive integer), then |p² - d·q²| < 2·√d + 1.
//
// The library's continued-fraction machinery (continued_fraction_convergents.ac)
// proves the adjacent-determinant identity p_{n+1}·q_n - p_n·q_{n+1} = ±1 and
// the denominator growth bounds, which are the algebraic core of the estimate.
// What is missing is the real-analysis statement that the continued fraction
// of √d converges to √d as a real number, so the estimate is stated here but
// not proved. The d = 2 instance is proved exactly above (Section 2): the
// convergent 3/2 satisfies |3² - 2·2²| = 1, and 1 < 2·√2 + 1.

// theorem pell_convergent_error_bound(d: Nat, p: Nat, q: Nat) {
//     ... |p² - d·q²| < 2·√d + 1 for convergents of the continued fraction of √d ...
// }

// Pell's equation has infinitely many solutions for every nonsquare d: the
// powers (x + y·√d)ⁿ of a nontrivial solution are again solutions, and the
// y-coordinates grow strictly, so the solutions are infinitely many.

// theorem pell_infinitely_many_solutions(d: Int) {
//     ...
// }
