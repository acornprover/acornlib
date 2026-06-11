from nat import Nat, double_addition_carry_count, mul_cancel_right, lte_trans
from combinatorics import binom, binom_pos, factorial_nonzero
from number_theory.factorisation import count_prime_factor, count_prime_factor_mul
from number_theory.falling_product import central_binom, falling_product, falling_product_nonzero,
    falling_product_mul_factorial_complement, falling_product_prime_count_sum,
    count_prime_factor_falling_product
from number_theory.falling_product_divisibility import falling_product_target_prime_bound,
    falling_product_target_primewise_bound, falling_product_divides_target_of_target_primewise_bound,
    falling_product_target_primewise_bound_of_divides,
    falling_product_central_binom_prime_bound, falling_product_central_binom_carry_bound,
    falling_product_central_binom_primewise_bound, falling_product_central_binom_carrywise_bound,
    falling_product_divides_central_binom_of_primewise_bound,
    falling_product_divides_central_binom_of_carrywise_bound,
    falling_product_central_binom_primewise_bound_of_divides,
    falling_product_central_binom_carrywise_bound_of_divides
from number_theory.legendre import prime_factor_count_upto, legendre_factorial

numerals Nat

/// A positive falling product is a binomial coefficient times a factorial.
theorem falling_product_eq_binom_mul_factorial(n: Nat, k: Nat) {
    k < n implies falling_product(n, k) = n.binom(k.suc) * k.suc.factorial
} by {
    if k < n {
        k.suc <= n
        not n < k.suc
        let c: Nat = (n - k.suc).factorial
        factorial_nonzero(n - k.suc)
        c != Nat.0
        falling_product_mul_factorial_complement(n, k)
        falling_product(n, k) * c = n.factorial
        n.binom(k.suc) * k.suc.factorial * c = n.factorial
        n.binom(k.suc) * k.suc.factorial * c =
            (n.binom(k.suc) * k.suc.factorial) * c
        falling_product(n, k) * c =
            (n.binom(k.suc) * k.suc.factorial) * c
        mul_cancel_right(c, falling_product(n, k),
            n.binom(k.suc) * k.suc.factorial)
        falling_product(n, k) = n.binom(k.suc) * k.suc.factorial
    }
}

/// The binomial-factorial product form of a positive falling product.
theorem binom_mul_factorial_eq_falling_product(n: Nat, k: Nat) {
    k < n implies n.binom(k.suc) * k.suc.factorial = falling_product(n, k)
} by {
    if k < n {
        falling_product_eq_binom_mul_factorial(n, k)
        falling_product(n, k) = n.binom(k.suc) * k.suc.factorial
        n.binom(k.suc) * k.suc.factorial = falling_product(n, k)
    }
}

/// The binomial coefficient in the falling-product identity is positive.
theorem falling_product_binom_factor_positive(n: Nat, k: Nat) {
    k < n implies Nat.0 < n.binom(k.suc)
} by {
    if k < n {
        k.suc <= n
        binom_pos(n, k.suc)
        Nat.0 < n.binom(k.suc)
    }
}

/// The binomial coefficient in the falling-product identity is nonzero.
theorem falling_product_binom_factor_ne_zero(n: Nat, k: Nat) {
    k < n implies n.binom(k.suc) != Nat.0
} by {
    if k < n {
        falling_product_binom_factor_positive(n, k)
        Nat.0 < n.binom(k.suc)
        n.binom(k.suc) != Nat.0
    }
}

/// The factorial in the falling-product identity is nonzero.
theorem falling_product_length_factorial_ne_zero(k: Nat) {
    k.suc.factorial != Nat.0
} by {
    factorial_nonzero(k.suc)
}

/// The prime valuation of a positive falling product splits into the valuation
/// of the associated binomial coefficient and the valuation of the factorial
/// length.
theorem count_prime_factor_falling_product_eq_binom_add_factorial(
    p: Nat, n: Nat, k: Nat
) {
    k < n implies
        count_prime_factor(p, falling_product(n, k)) =
            count_prime_factor(p, n.binom(k.suc)) +
            count_prime_factor(p, k.suc.factorial)
} by {
    if k < n {
        falling_product_eq_binom_mul_factorial(n, k)
        falling_product(n, k) = n.binom(k.suc) * k.suc.factorial
        falling_product_binom_factor_ne_zero(n, k)
        n.binom(k.suc) != Nat.0
        falling_product_length_factorial_ne_zero(k)
        k.suc.factorial != Nat.0
        count_prime_factor_mul(p, n.binom(k.suc), k.suc.factorial)
        count_prime_factor(p, n.binom(k.suc) * k.suc.factorial) =
            count_prime_factor(p, n.binom(k.suc)) +
            count_prime_factor(p, k.suc.factorial)
        count_prime_factor(p, falling_product(n, k)) =
            count_prime_factor(p, n.binom(k.suc)) +
            count_prime_factor(p, k.suc.factorial)
    }
}

/// The falling-product valuation sum is the sum of the binomial valuation and
/// the factorial valuation of the length.
theorem falling_product_prime_count_sum_eq_binom_add_factorial(
    p: Nat, n: Nat, k: Nat
) {
    k < n implies
        falling_product_prime_count_sum(p, n, k) =
            count_prime_factor(p, n.binom(k.suc)) +
            count_prime_factor(p, k.suc.factorial)
} by {
    if k < n {
        count_prime_factor_falling_product(p, n, k)
        count_prime_factor(p, falling_product(n, k)) =
            falling_product_prime_count_sum(p, n, k)
        count_prime_factor_falling_product_eq_binom_add_factorial(p, n, k)
        count_prime_factor(p, falling_product(n, k)) =
            count_prime_factor(p, n.binom(k.suc)) +
            count_prime_factor(p, k.suc.factorial)
        falling_product_prime_count_sum(p, n, k) =
            count_prime_factor(p, n.binom(k.suc)) +
            count_prime_factor(p, k.suc.factorial)
    }
}

/// The falling-product valuation sum is the sum of the associated binomial
/// valuation and the Legendre sum for the factorial length.
theorem falling_product_prime_count_sum_eq_binom_add_upto(
    p: Nat, n: Nat, k: Nat
) {
    k < n implies
        falling_product_prime_count_sum(p, n, k) =
            count_prime_factor(p, n.binom(k.suc)) +
            prime_factor_count_upto(p, k.suc)
} by {
    if k < n {
        falling_product_prime_count_sum_eq_binom_add_factorial(p, n, k)
        falling_product_prime_count_sum(p, n, k) =
            count_prime_factor(p, n.binom(k.suc)) +
            count_prime_factor(p, k.suc.factorial)
        legendre_factorial(p, k.suc)
        count_prime_factor(p, k.suc.factorial) =
            prime_factor_count_upto(p, k.suc)
        falling_product_prime_count_sum(p, n, k) =
            count_prime_factor(p, n.binom(k.suc)) +
            prime_factor_count_upto(p, k.suc)
    }
}

/// The associated binomial valuation plus the factorial valuation of the
/// length is the falling-product valuation sum.
theorem binom_add_factorial_eq_falling_product_prime_count_sum(
    p: Nat, n: Nat, k: Nat
) {
    k < n implies
        count_prime_factor(p, n.binom(k.suc)) +
        count_prime_factor(p, k.suc.factorial) =
            falling_product_prime_count_sum(p, n, k)
} by {
    if k < n {
        falling_product_prime_count_sum_eq_binom_add_factorial(p, n, k)
        falling_product_prime_count_sum(p, n, k) =
            count_prime_factor(p, n.binom(k.suc)) +
            count_prime_factor(p, k.suc.factorial)
        count_prime_factor(p, n.binom(k.suc)) +
            count_prime_factor(p, k.suc.factorial) =
                falling_product_prime_count_sum(p, n, k)
    }
}

/// The associated binomial valuation plus the Legendre sum for the factorial
/// length is the falling-product valuation sum.
theorem binom_add_upto_eq_falling_product_prime_count_sum(
    p: Nat, n: Nat, k: Nat
) {
    k < n implies
        count_prime_factor(p, n.binom(k.suc)) +
        prime_factor_count_upto(p, k.suc) =
            falling_product_prime_count_sum(p, n, k)
} by {
    if k < n {
        falling_product_prime_count_sum_eq_binom_add_upto(p, n, k)
        falling_product_prime_count_sum(p, n, k) =
            count_prime_factor(p, n.binom(k.suc)) +
            prime_factor_count_upto(p, k.suc)
        count_prime_factor(p, n.binom(k.suc)) +
            prime_factor_count_upto(p, k.suc) =
                falling_product_prime_count_sum(p, n, k)
    }
}

/// The prime valuation of a positive falling product plus the valuation of the
/// complementary factorial is the valuation of the top factorial.
theorem count_prime_factor_falling_product_add_complement_factorial(
    p: Nat, n: Nat, k: Nat
) {
    k < n implies
        count_prime_factor(p, falling_product(n, k)) +
        count_prime_factor(p, (n - k.suc).factorial) =
            count_prime_factor(p, n.factorial)
} by {
    if k < n {
        falling_product_nonzero(n, k)
        falling_product(n, k) != Nat.0
        factorial_nonzero(n - k.suc)
        (n - k.suc).factorial != Nat.0
        count_prime_factor_mul(p, falling_product(n, k), (n - k.suc).factorial)
        count_prime_factor(p, falling_product(n, k) * (n - k.suc).factorial) =
            count_prime_factor(p, falling_product(n, k)) +
            count_prime_factor(p, (n - k.suc).factorial)
        falling_product_mul_factorial_complement(n, k)
        falling_product(n, k) * (n - k.suc).factorial = n.factorial
        count_prime_factor(p, n.factorial) =
            count_prime_factor(p, falling_product(n, k)) +
            count_prime_factor(p, (n - k.suc).factorial)
        count_prime_factor(p, falling_product(n, k)) +
            count_prime_factor(p, (n - k.suc).factorial) =
                count_prime_factor(p, n.factorial)
    }
}

/// The valuation of the complementary factorial plus the prime valuation of a
/// positive falling product is the valuation of the top factorial.
theorem complement_factorial_add_count_prime_factor_falling_product(
    p: Nat, n: Nat, k: Nat
) {
    k < n implies
        count_prime_factor(p, (n - k.suc).factorial) +
        count_prime_factor(p, falling_product(n, k)) =
            count_prime_factor(p, n.factorial)
} by {
    if k < n {
        count_prime_factor_falling_product_add_complement_factorial(p, n, k)
        count_prime_factor(p, falling_product(n, k)) +
            count_prime_factor(p, (n - k.suc).factorial) =
                count_prime_factor(p, n.factorial)
        count_prime_factor(p, (n - k.suc).factorial) +
            count_prime_factor(p, falling_product(n, k)) =
                count_prime_factor(p, n.factorial)
    }
}

/// The falling-product valuation sum plus the valuation of the complementary
/// factorial is the valuation of the top factorial.
theorem falling_product_prime_count_sum_add_complement_factorial(
    p: Nat, n: Nat, k: Nat
) {
    k < n implies
        falling_product_prime_count_sum(p, n, k) +
        count_prime_factor(p, (n - k.suc).factorial) =
            count_prime_factor(p, n.factorial)
} by {
    if k < n {
        count_prime_factor_falling_product(p, n, k)
        count_prime_factor(p, falling_product(n, k)) =
            falling_product_prime_count_sum(p, n, k)
        count_prime_factor_falling_product_add_complement_factorial(p, n, k)
        count_prime_factor(p, falling_product(n, k)) +
            count_prime_factor(p, (n - k.suc).factorial) =
                count_prime_factor(p, n.factorial)
        falling_product_prime_count_sum(p, n, k) +
            count_prime_factor(p, (n - k.suc).factorial) =
                count_prime_factor(p, n.factorial)
    }
}

/// The complementary factorial valuation plus the falling-product valuation
/// sum is the valuation of the top factorial.
theorem complement_factorial_add_falling_product_prime_count_sum(
    p: Nat, n: Nat, k: Nat
) {
    k < n implies
        count_prime_factor(p, (n - k.suc).factorial) +
        falling_product_prime_count_sum(p, n, k) =
            count_prime_factor(p, n.factorial)
} by {
    if k < n {
        falling_product_prime_count_sum_add_complement_factorial(p, n, k)
        falling_product_prime_count_sum(p, n, k) +
            count_prime_factor(p, (n - k.suc).factorial) =
                count_prime_factor(p, n.factorial)
        count_prime_factor(p, (n - k.suc).factorial) +
            falling_product_prime_count_sum(p, n, k) =
                count_prime_factor(p, n.factorial)
    }
}

/// The falling-product valuation sum plus the Legendre sum below the interval
/// is the Legendre sum up to the top.
theorem falling_product_prime_count_sum_add_complement_upto(
    p: Nat, n: Nat, k: Nat
) {
    k < n implies
        falling_product_prime_count_sum(p, n, k) +
        prime_factor_count_upto(p, n - k.suc) =
            prime_factor_count_upto(p, n)
} by {
    if k < n {
        falling_product_prime_count_sum_add_complement_factorial(p, n, k)
        falling_product_prime_count_sum(p, n, k) +
            count_prime_factor(p, (n - k.suc).factorial) =
                count_prime_factor(p, n.factorial)
        legendre_factorial(p, n - k.suc)
        count_prime_factor(p, (n - k.suc).factorial) =
            prime_factor_count_upto(p, n - k.suc)
        legendre_factorial(p, n)
        count_prime_factor(p, n.factorial) = prime_factor_count_upto(p, n)
        falling_product_prime_count_sum(p, n, k) +
            prime_factor_count_upto(p, n - k.suc) =
                prime_factor_count_upto(p, n)
    }
}

/// The Legendre sum below the interval plus the falling-product valuation sum
/// is the Legendre sum up to the top.
theorem complement_upto_add_falling_product_prime_count_sum(
    p: Nat, n: Nat, k: Nat
) {
    k < n implies
        prime_factor_count_upto(p, n - k.suc) +
        falling_product_prime_count_sum(p, n, k) =
            prime_factor_count_upto(p, n)
} by {
    if k < n {
        falling_product_prime_count_sum_add_complement_upto(p, n, k)
        falling_product_prime_count_sum(p, n, k) +
            prime_factor_count_upto(p, n - k.suc) =
                prime_factor_count_upto(p, n)
        prime_factor_count_upto(p, n - k.suc) +
            falling_product_prime_count_sum(p, n, k) =
                prime_factor_count_upto(p, n)
    }
}

/// The binomial valuation, length Legendre sum, and complementary Legendre sum
/// together form the Legendre sum up to the top.
theorem binom_add_length_upto_add_complement_upto(
    p: Nat, n: Nat, k: Nat
) {
    k < n implies
        count_prime_factor(p, n.binom(k.suc)) +
        prime_factor_count_upto(p, k.suc) +
        prime_factor_count_upto(p, n - k.suc) =
            prime_factor_count_upto(p, n)
} by {
    if k < n {
        falling_product_prime_count_sum_eq_binom_add_upto(p, n, k)
        falling_product_prime_count_sum(p, n, k) =
            count_prime_factor(p, n.binom(k.suc)) +
            prime_factor_count_upto(p, k.suc)
        falling_product_prime_count_sum_add_complement_upto(p, n, k)
        falling_product_prime_count_sum(p, n, k) +
            prime_factor_count_upto(p, n - k.suc) =
                prime_factor_count_upto(p, n)
        count_prime_factor(p, n.binom(k.suc)) +
            prime_factor_count_upto(p, k.suc) +
            prime_factor_count_upto(p, n - k.suc) =
                prime_factor_count_upto(p, n)
    }
}

/// The complementary Legendre sum plus the binomial valuation and length
/// Legendre sum form the Legendre sum up to the top.
theorem complement_upto_add_binom_add_length_upto(
    p: Nat, n: Nat, k: Nat
) {
    k < n implies
        prime_factor_count_upto(p, n - k.suc) +
        (count_prime_factor(p, n.binom(k.suc)) +
            prime_factor_count_upto(p, k.suc)) =
                prime_factor_count_upto(p, n)
} by {
    if k < n {
        binom_add_length_upto_add_complement_upto(p, n, k)
        count_prime_factor(p, n.binom(k.suc)) +
            prime_factor_count_upto(p, k.suc) +
            prime_factor_count_upto(p, n - k.suc) =
                prime_factor_count_upto(p, n)
        prime_factor_count_upto(p, n - k.suc) +
            (count_prime_factor(p, n.binom(k.suc)) +
                prime_factor_count_upto(p, k.suc)) =
                    prime_factor_count_upto(p, n)
    }
}

/// The binomial valuation plus the factorial valuations of the chosen length
/// and complementary length is the valuation of the top factorial.
theorem binom_add_factorial_add_complement_factorial(
    p: Nat, n: Nat, k: Nat
) {
    k < n implies
        count_prime_factor(p, n.binom(k.suc)) +
        count_prime_factor(p, k.suc.factorial) +
        count_prime_factor(p, (n - k.suc).factorial) =
            count_prime_factor(p, n.factorial)
} by {
    if k < n {
        falling_product_prime_count_sum_eq_binom_add_factorial(p, n, k)
        falling_product_prime_count_sum(p, n, k) =
            count_prime_factor(p, n.binom(k.suc)) +
            count_prime_factor(p, k.suc.factorial)
        falling_product_prime_count_sum_add_complement_factorial(p, n, k)
        falling_product_prime_count_sum(p, n, k) +
            count_prime_factor(p, (n - k.suc).factorial) =
                count_prime_factor(p, n.factorial)
        count_prime_factor(p, n.binom(k.suc)) +
            count_prime_factor(p, k.suc.factorial) +
            count_prime_factor(p, (n - k.suc).factorial) =
                count_prime_factor(p, n.factorial)
    }
}

/// The associated binomial coefficient divides a positive falling product.
theorem binom_divides_falling_product(n: Nat, k: Nat) {
    k < n implies n.binom(k.suc).divides(falling_product(n, k))
} by {
    if k < n {
        falling_product_eq_binom_mul_factorial(n, k)
        falling_product(n, k) = n.binom(k.suc) * k.suc.factorial
        n.binom(k.suc).divides(falling_product(n, k))
    }
}

/// The factorial of the length divides a positive falling product.
theorem length_factorial_divides_falling_product(n: Nat, k: Nat) {
    k < n implies k.suc.factorial.divides(falling_product(n, k))
} by {
    if k < n {
        falling_product_eq_binom_mul_factorial(n, k)
        falling_product(n, k) = n.binom(k.suc) * k.suc.factorial
        k.suc.factorial * n.binom(k.suc) = n.binom(k.suc) * k.suc.factorial
        k.suc.factorial.divides(falling_product(n, k))
    }
}

/// The valuation of the associated binomial coefficient is bounded by the
/// falling-product valuation sum.
theorem count_prime_factor_binom_le_falling_product_prime_count_sum(
    p: Nat, n: Nat, k: Nat
) {
    k < n implies
        count_prime_factor(p, n.binom(k.suc)) <= falling_product_prime_count_sum(p, n, k)
} by {
    if k < n {
        let a: Nat = count_prime_factor(p, n.binom(k.suc))
        let b: Nat = count_prime_factor(p, k.suc.factorial)
        let s: Nat = falling_product_prime_count_sum(p, n, k)
        falling_product_prime_count_sum_eq_binom_add_factorial(p, n, k)
        s = a + b
        a + b = s
        a <= s
        count_prime_factor(p, n.binom(k.suc)) <= falling_product_prime_count_sum(p, n, k)
    }
}

/// The valuation of the factorial length is bounded by the falling-product
/// valuation sum.
theorem count_prime_factor_length_factorial_le_falling_product_prime_count_sum(
    p: Nat, n: Nat, k: Nat
) {
    k < n implies
        count_prime_factor(p, k.suc.factorial) <= falling_product_prime_count_sum(p, n, k)
} by {
    if k < n {
        let a: Nat = count_prime_factor(p, n.binom(k.suc))
        let b: Nat = count_prime_factor(p, k.suc.factorial)
        let s: Nat = falling_product_prime_count_sum(p, n, k)
        falling_product_prime_count_sum_eq_binom_add_factorial(p, n, k)
        s = a + b
        b + a = a + b
        b + a = s
        b <= s
        count_prime_factor(p, k.suc.factorial) <= falling_product_prime_count_sum(p, n, k)
    }
}

/// The Legendre sum for the length is bounded by the falling-product valuation
/// sum.
theorem prime_factor_count_upto_length_le_falling_product_prime_count_sum(
    p: Nat, n: Nat, k: Nat
) {
    k < n implies
        prime_factor_count_upto(p, k.suc) <= falling_product_prime_count_sum(p, n, k)
} by {
    if k < n {
        let a: Nat = count_prime_factor(p, n.binom(k.suc))
        let b: Nat = prime_factor_count_upto(p, k.suc)
        let s: Nat = falling_product_prime_count_sum(p, n, k)
        falling_product_prime_count_sum_eq_binom_add_upto(p, n, k)
        s = a + b
        b + a = a + b
        b + a = s
        b <= s
        prime_factor_count_upto(p, k.suc) <= falling_product_prime_count_sum(p, n, k)
    }
}

/// The associated binomial valuation is bounded by the Legendre sum up to the
/// top.
theorem count_prime_factor_binom_le_prime_factor_count_upto_top(
    p: Nat, n: Nat, k: Nat
) {
    k < n implies
        count_prime_factor(p, n.binom(k.suc)) <= prime_factor_count_upto(p, n)
} by {
    if k < n {
        let a: Nat = count_prime_factor(p, n.binom(k.suc))
        let s: Nat = falling_product_prime_count_sum(p, n, k)
        let top: Nat = prime_factor_count_upto(p, n)
        count_prime_factor_binom_le_falling_product_prime_count_sum(p, n, k)
        a <= s
        falling_product_prime_count_sum_add_complement_upto(p, n, k)
        s + prime_factor_count_upto(p, n - k.suc) = top
        s <= top
        lte_trans(a, s, top)
        a <= top
        count_prime_factor(p, n.binom(k.suc)) <= prime_factor_count_upto(p, n)
    }
}

/// The falling-product valuation sum is bounded by the Legendre sum up to the
/// top.
theorem falling_product_prime_count_sum_le_prime_factor_count_upto_top(
    p: Nat, n: Nat, k: Nat
) {
    k < n implies
        falling_product_prime_count_sum(p, n, k) <= prime_factor_count_upto(p, n)
} by {
    if k < n {
        let s: Nat = falling_product_prime_count_sum(p, n, k)
        let c: Nat = prime_factor_count_upto(p, n - k.suc)
        let top: Nat = prime_factor_count_upto(p, n)
        falling_product_prime_count_sum_add_complement_upto(p, n, k)
        s + c = top
        s <= top
        falling_product_prime_count_sum(p, n, k) <= prime_factor_count_upto(p, n)
    }
}

/// The Legendre sum below the interval is bounded by the Legendre sum up to
/// the top.
theorem complement_upto_le_prime_factor_count_upto_top(
    p: Nat, n: Nat, k: Nat
) {
    k < n implies prime_factor_count_upto(p, n - k.suc) <= prime_factor_count_upto(p, n)
} by {
    if k < n {
        let s: Nat = falling_product_prime_count_sum(p, n, k)
        let c: Nat = prime_factor_count_upto(p, n - k.suc)
        let top: Nat = prime_factor_count_upto(p, n)
        complement_upto_add_falling_product_prime_count_sum(p, n, k)
        c + s = top
        c <= top
        prime_factor_count_upto(p, n - k.suc) <= prime_factor_count_upto(p, n)
    }
}

/// The binomial-Legendre falling-product bound is bounded by the valuation of
/// a target natural at a prime.
define falling_product_binomial_target_prime_bound(
    p: Nat, n: Nat, k: Nat, target: Nat
) -> Bool {
    count_prime_factor(p, n.binom(k.suc)) +
        prime_factor_count_upto(p, k.suc) <= count_prime_factor(p, target)
}

/// Every prime binomial-Legendre falling-product bound is bounded by the
/// matching valuation of a target natural.
define falling_product_binomial_target_primewise_bound(
    n: Nat, k: Nat, target: Nat
) -> Bool {
    forall(p: Nat) {
        p.is_prime implies falling_product_binomial_target_prime_bound(p, n, k, target)
    }
}

/// The binomial-Legendre falling-product bound is bounded by the central
/// binomial valuation at a prime.
define falling_product_binomial_central_binom_prime_bound(
    p: Nat, n: Nat, k: Nat
) -> Bool {
    count_prime_factor(p, n.binom(k.suc)) +
        prime_factor_count_upto(p, k.suc) <= count_prime_factor(p, central_binom(n))
}

/// The binomial-Legendre falling-product bound is bounded by the central
/// binomial carry count at a prime base.
define falling_product_binomial_central_binom_carry_bound(
    p: Nat, n: Nat, k: Nat
) -> Bool {
    count_prime_factor(p, n.binom(k.suc)) +
        prime_factor_count_upto(p, k.suc) <= double_addition_carry_count(p, n)
}

/// Every prime binomial-Legendre falling-product bound is bounded by the
/// matching central binomial valuation.
define falling_product_binomial_central_binom_primewise_bound(n: Nat, k: Nat) -> Bool {
    forall(p: Nat) {
        p.is_prime implies falling_product_binomial_central_binom_prime_bound(p, n, k)
    }
}

/// Every prime binomial-Legendre falling-product bound is bounded by the
/// matching central binomial carry count.
define falling_product_binomial_central_binom_carrywise_bound(n: Nat, k: Nat) -> Bool {
    forall(p: Nat) {
        p.is_prime implies falling_product_binomial_central_binom_carry_bound(p, n, k)
    }
}

/// A primewise binomial-Legendre target bound gives the target bound at any
/// particular prime.
theorem falling_product_binomial_target_prime_bound_of_primewise_bound(
    p: Nat, n: Nat, k: Nat, target: Nat
) {
    p.is_prime and falling_product_binomial_target_primewise_bound(n, k, target)
        implies falling_product_binomial_target_prime_bound(p, n, k, target)
} by {
    if p.is_prime and falling_product_binomial_target_primewise_bound(n, k, target) {
        let h: Bool = p.is_prime implies falling_product_binomial_target_prime_bound(p, n, k, target)
        h
        falling_product_binomial_target_prime_bound(p, n, k, target)
    }
}

/// A primewise binomial-Legendre central-binomial valuation bound gives the
/// valuation bound at any particular prime.
theorem falling_product_binomial_central_binom_prime_bound_of_primewise_bound(
    p: Nat, n: Nat, k: Nat
) {
    p.is_prime and falling_product_binomial_central_binom_primewise_bound(n, k)
        implies falling_product_binomial_central_binom_prime_bound(p, n, k)
} by {
    if p.is_prime and falling_product_binomial_central_binom_primewise_bound(n, k) {
        let h: Bool = p.is_prime implies falling_product_binomial_central_binom_prime_bound(p, n, k)
        h
        falling_product_binomial_central_binom_prime_bound(p, n, k)
    }
}

/// A primewise binomial-Legendre central-binomial carry bound gives the carry
/// bound at any particular prime.
theorem falling_product_binomial_central_binom_carry_bound_of_carrywise_bound(
    p: Nat, n: Nat, k: Nat
) {
    p.is_prime and falling_product_binomial_central_binom_carrywise_bound(n, k)
        implies falling_product_binomial_central_binom_carry_bound(p, n, k)
} by {
    if p.is_prime and falling_product_binomial_central_binom_carrywise_bound(n, k) {
        let h: Bool = p.is_prime implies falling_product_binomial_central_binom_carry_bound(p, n, k)
        h
        falling_product_binomial_central_binom_carry_bound(p, n, k)
    }
}

/// A positive falling-product target bound implies the binomial-Legendre target
/// bound.
theorem falling_product_binomial_target_prime_bound_of_target_prime_bound(
    p: Nat, n: Nat, k: Nat, target: Nat
) {
    k < n and falling_product_target_prime_bound(p, n, k, target)
        implies falling_product_binomial_target_prime_bound(p, n, k, target)
} by {
    if k < n and falling_product_target_prime_bound(p, n, k, target) {
        falling_product_prime_count_sum_eq_binom_add_upto(p, n, k)
        falling_product_prime_count_sum(p, n, k) =
            count_prime_factor(p, n.binom(k.suc)) +
            prime_factor_count_upto(p, k.suc)
        falling_product_prime_count_sum(p, n, k) <= count_prime_factor(p, target)
        count_prime_factor(p, n.binom(k.suc)) +
            prime_factor_count_upto(p, k.suc) <= count_prime_factor(p, target)
        falling_product_binomial_target_prime_bound(p, n, k, target)
    }
}

/// A positive binomial-Legendre target bound implies the falling-product target
/// bound.
theorem falling_product_target_prime_bound_of_binomial_target_prime_bound(
    p: Nat, n: Nat, k: Nat, target: Nat
) {
    k < n and falling_product_binomial_target_prime_bound(p, n, k, target)
        implies falling_product_target_prime_bound(p, n, k, target)
} by {
    if k < n and falling_product_binomial_target_prime_bound(p, n, k, target) {
        falling_product_prime_count_sum_eq_binom_add_upto(p, n, k)
        falling_product_prime_count_sum(p, n, k) =
            count_prime_factor(p, n.binom(k.suc)) +
            prime_factor_count_upto(p, k.suc)
        count_prime_factor(p, n.binom(k.suc)) +
            prime_factor_count_upto(p, k.suc) <= count_prime_factor(p, target)
        falling_product_prime_count_sum(p, n, k) <= count_prime_factor(p, target)
        falling_product_target_prime_bound(p, n, k, target)
    }
}

/// Positive falling-product target bounds imply the corresponding
/// binomial-Legendre primewise target bounds.
theorem falling_product_binomial_target_primewise_bound_of_target_primewise_bound(
    n: Nat, k: Nat, target: Nat
) {
    k < n and falling_product_target_primewise_bound(n, k, target)
        implies falling_product_binomial_target_primewise_bound(n, k, target)
} by {
    if k < n and falling_product_target_primewise_bound(n, k, target) {
        forall(p: Nat) {
            if p.is_prime {
                let h: Bool = p.is_prime implies falling_product_target_prime_bound(p, n, k, target)
                h
                falling_product_target_prime_bound(p, n, k, target)
                falling_product_binomial_target_prime_bound_of_target_prime_bound(
                    p, n, k, target)
                falling_product_binomial_target_prime_bound(p, n, k, target)
            }
        }
        falling_product_binomial_target_primewise_bound(n, k, target)
    }
}

/// Positive binomial-Legendre primewise target bounds imply falling-product
/// target bounds.
theorem falling_product_target_primewise_bound_of_binomial_target_primewise_bound(
    n: Nat, k: Nat, target: Nat
) {
    k < n and falling_product_binomial_target_primewise_bound(n, k, target)
        implies falling_product_target_primewise_bound(n, k, target)
} by {
    if k < n and falling_product_binomial_target_primewise_bound(n, k, target) {
        forall(p: Nat) {
            if p.is_prime {
                falling_product_binomial_target_prime_bound_of_primewise_bound(
                    p, n, k, target)
                falling_product_binomial_target_prime_bound(p, n, k, target)
                falling_product_target_prime_bound_of_binomial_target_prime_bound(
                    p, n, k, target)
                falling_product_target_prime_bound(p, n, k, target)
            }
        }
        falling_product_target_primewise_bound(n, k, target)
    }
}

/// A nonzero target is divisible by a positive falling product whenever the
/// binomial-Legendre primewise target bounds hold.
theorem falling_product_divides_target_of_binomial_target_primewise_bound(
    n: Nat, k: Nat, target: Nat
) {
    k < n and target != Nat.0 and
    falling_product_binomial_target_primewise_bound(n, k, target)
        implies falling_product(n, k).divides(target)
} by {
    if k < n and target != Nat.0 and
    falling_product_binomial_target_primewise_bound(n, k, target) {
        falling_product_target_primewise_bound_of_binomial_target_primewise_bound(
            n, k, target)
        falling_product_target_primewise_bound(n, k, target)
        falling_product_divides_target_of_target_primewise_bound(n, k, target)
        falling_product(n, k).divides(target)
    }
}

/// Divisibility of a nonzero target by a positive falling product implies the
/// binomial-Legendre primewise target bounds.
theorem falling_product_binomial_target_primewise_bound_of_divides(
    n: Nat, k: Nat, target: Nat
) {
    k < n and target != Nat.0 and falling_product(n, k).divides(target)
        implies falling_product_binomial_target_primewise_bound(n, k, target)
} by {
    if k < n and target != Nat.0 and falling_product(n, k).divides(target) {
        falling_product_target_primewise_bound_of_divides(n, k, target)
        falling_product_target_primewise_bound(n, k, target)
        falling_product_binomial_target_primewise_bound_of_target_primewise_bound(
            n, k, target)
        falling_product_binomial_target_primewise_bound(n, k, target)
    }
}

/// A positive falling product divides a nonzero target iff the
/// binomial-Legendre primewise target bounds hold.
theorem falling_product_divides_target_iff_binomial_target_primewise_bound(
    n: Nat, k: Nat, target: Nat
) {
    k < n and target != Nat.0 implies (
        falling_product(n, k).divides(target) =
        falling_product_binomial_target_primewise_bound(n, k, target)
    )
} by {
    if k < n and target != Nat.0 {
        let divides_target: Bool = falling_product(n, k).divides(target)
        let binomial_bound: Bool = falling_product_binomial_target_primewise_bound(
            n, k, target)
        if divides_target {
            falling_product_binomial_target_primewise_bound_of_divides(
                n, k, target)
            binomial_bound
        }
        if binomial_bound {
            falling_product_divides_target_of_binomial_target_primewise_bound(
                n, k, target)
            divides_target
        }
        divides_target = binomial_bound
        falling_product(n, k).divides(target) =
            falling_product_binomial_target_primewise_bound(n, k, target)
    }
}

/// A positive falling-product central-binomial valuation bound implies the
/// binomial-Legendre central-binomial valuation bound.
theorem falling_product_binomial_central_binom_prime_bound_of_prime_bound(
    p: Nat, n: Nat, k: Nat
) {
    k < n and falling_product_central_binom_prime_bound(p, n, k)
        implies falling_product_binomial_central_binom_prime_bound(p, n, k)
} by {
    if k < n and falling_product_central_binom_prime_bound(p, n, k) {
        falling_product_prime_count_sum_eq_binom_add_upto(p, n, k)
        falling_product_prime_count_sum(p, n, k) =
            count_prime_factor(p, n.binom(k.suc)) +
            prime_factor_count_upto(p, k.suc)
        falling_product_prime_count_sum(p, n, k) <= count_prime_factor(p, central_binom(n))
        count_prime_factor(p, n.binom(k.suc)) +
            prime_factor_count_upto(p, k.suc) <= count_prime_factor(p, central_binom(n))
        falling_product_binomial_central_binom_prime_bound(p, n, k)
    }
}

/// A positive binomial-Legendre central-binomial valuation bound implies the
/// falling-product central-binomial valuation bound.
theorem falling_product_central_binom_prime_bound_of_binomial_prime_bound(
    p: Nat, n: Nat, k: Nat
) {
    k < n and falling_product_binomial_central_binom_prime_bound(p, n, k)
        implies falling_product_central_binom_prime_bound(p, n, k)
} by {
    if k < n and falling_product_binomial_central_binom_prime_bound(p, n, k) {
        falling_product_prime_count_sum_eq_binom_add_upto(p, n, k)
        falling_product_prime_count_sum(p, n, k) =
            count_prime_factor(p, n.binom(k.suc)) +
            prime_factor_count_upto(p, k.suc)
        count_prime_factor(p, n.binom(k.suc)) +
            prime_factor_count_upto(p, k.suc) <= count_prime_factor(p, central_binom(n))
        falling_product_prime_count_sum(p, n, k) <= count_prime_factor(p, central_binom(n))
        falling_product_central_binom_prime_bound(p, n, k)
    }
}

/// A positive falling-product central-binomial carry bound implies the
/// binomial-Legendre central-binomial carry bound.
theorem falling_product_binomial_central_binom_carry_bound_of_carry_bound(
    p: Nat, n: Nat, k: Nat
) {
    k < n and falling_product_central_binom_carry_bound(p, n, k)
        implies falling_product_binomial_central_binom_carry_bound(p, n, k)
} by {
    if k < n and falling_product_central_binom_carry_bound(p, n, k) {
        falling_product_prime_count_sum_eq_binom_add_upto(p, n, k)
        falling_product_prime_count_sum(p, n, k) =
            count_prime_factor(p, n.binom(k.suc)) +
            prime_factor_count_upto(p, k.suc)
        falling_product_prime_count_sum(p, n, k) <= double_addition_carry_count(p, n)
        count_prime_factor(p, n.binom(k.suc)) +
            prime_factor_count_upto(p, k.suc) <= double_addition_carry_count(p, n)
        falling_product_binomial_central_binom_carry_bound(p, n, k)
    }
}

/// A positive binomial-Legendre central-binomial carry bound implies the
/// falling-product central-binomial carry bound.
theorem falling_product_central_binom_carry_bound_of_binomial_carry_bound(
    p: Nat, n: Nat, k: Nat
) {
    k < n and falling_product_binomial_central_binom_carry_bound(p, n, k)
        implies falling_product_central_binom_carry_bound(p, n, k)
} by {
    if k < n and falling_product_binomial_central_binom_carry_bound(p, n, k) {
        falling_product_prime_count_sum_eq_binom_add_upto(p, n, k)
        falling_product_prime_count_sum(p, n, k) =
            count_prime_factor(p, n.binom(k.suc)) +
            prime_factor_count_upto(p, k.suc)
        count_prime_factor(p, n.binom(k.suc)) +
            prime_factor_count_upto(p, k.suc) <= double_addition_carry_count(p, n)
        falling_product_prime_count_sum(p, n, k) <= double_addition_carry_count(p, n)
        falling_product_central_binom_carry_bound(p, n, k)
    }
}

/// Positive central-binomial primewise bounds imply the binomial-Legendre
/// central-binomial primewise bounds.
theorem falling_product_binomial_central_binom_primewise_bound_of_primewise_bound(
    n: Nat, k: Nat
) {
    k < n and falling_product_central_binom_primewise_bound(n, k)
        implies falling_product_binomial_central_binom_primewise_bound(n, k)
} by {
    if k < n and falling_product_central_binom_primewise_bound(n, k) {
        forall(p: Nat) {
            if p.is_prime {
                let h: Bool = p.is_prime implies falling_product_central_binom_prime_bound(p, n, k)
                h
                falling_product_central_binom_prime_bound(p, n, k)
                falling_product_binomial_central_binom_prime_bound_of_prime_bound(
                    p, n, k)
                falling_product_binomial_central_binom_prime_bound(p, n, k)
            }
        }
        falling_product_binomial_central_binom_primewise_bound(n, k)
    }
}

/// Positive binomial-Legendre central-binomial primewise bounds imply the
/// central-binomial primewise bounds.
theorem falling_product_central_binom_primewise_bound_of_binomial_primewise_bound(
    n: Nat, k: Nat
) {
    k < n and falling_product_binomial_central_binom_primewise_bound(n, k)
        implies falling_product_central_binom_primewise_bound(n, k)
} by {
    if k < n and falling_product_binomial_central_binom_primewise_bound(n, k) {
        forall(p: Nat) {
            if p.is_prime {
                falling_product_binomial_central_binom_prime_bound_of_primewise_bound(
                    p, n, k)
                falling_product_binomial_central_binom_prime_bound(p, n, k)
                falling_product_central_binom_prime_bound_of_binomial_prime_bound(
                    p, n, k)
                falling_product_central_binom_prime_bound(p, n, k)
            }
        }
        falling_product_central_binom_primewise_bound(n, k)
    }
}

/// Positive central-binomial carrywise bounds imply the binomial-Legendre
/// central-binomial carrywise bounds.
theorem falling_product_binomial_central_binom_carrywise_bound_of_carrywise_bound(
    n: Nat, k: Nat
) {
    k < n and falling_product_central_binom_carrywise_bound(n, k)
        implies falling_product_binomial_central_binom_carrywise_bound(n, k)
} by {
    if k < n and falling_product_central_binom_carrywise_bound(n, k) {
        forall(p: Nat) {
            if p.is_prime {
                let h: Bool = p.is_prime implies falling_product_central_binom_carry_bound(p, n, k)
                h
                falling_product_central_binom_carry_bound(p, n, k)
                falling_product_binomial_central_binom_carry_bound_of_carry_bound(
                    p, n, k)
                falling_product_binomial_central_binom_carry_bound(p, n, k)
            }
        }
        falling_product_binomial_central_binom_carrywise_bound(n, k)
    }
}

/// Positive binomial-Legendre central-binomial carrywise bounds imply the
/// central-binomial carrywise bounds.
theorem falling_product_central_binom_carrywise_bound_of_binomial_carrywise_bound(
    n: Nat, k: Nat
) {
    k < n and falling_product_binomial_central_binom_carrywise_bound(n, k)
        implies falling_product_central_binom_carrywise_bound(n, k)
} by {
    if k < n and falling_product_binomial_central_binom_carrywise_bound(n, k) {
        forall(p: Nat) {
            if p.is_prime {
                falling_product_binomial_central_binom_carry_bound_of_carrywise_bound(
                    p, n, k)
                falling_product_binomial_central_binom_carry_bound(p, n, k)
                falling_product_central_binom_carry_bound_of_binomial_carry_bound(
                    p, n, k)
                falling_product_central_binom_carry_bound(p, n, k)
            }
        }
        falling_product_central_binom_carrywise_bound(n, k)
    }
}

/// Binomial-Legendre central-binomial primewise bounds imply divisibility of
/// the central binomial coefficient by the falling product.
theorem falling_product_divides_central_binom_of_binomial_primewise_bound(
    n: Nat, k: Nat
) {
    k < n and falling_product_binomial_central_binom_primewise_bound(n, k)
        implies falling_product(n, k).divides(central_binom(n))
} by {
    if k < n and falling_product_binomial_central_binom_primewise_bound(n, k) {
        falling_product_central_binom_primewise_bound_of_binomial_primewise_bound(n, k)
        falling_product_central_binom_primewise_bound(n, k)
        falling_product_divides_central_binom_of_primewise_bound(n, k)
        falling_product(n, k).divides(central_binom(n))
    }
}

/// Binomial-Legendre central-binomial carrywise bounds imply divisibility of
/// the central binomial coefficient by the falling product.
theorem falling_product_divides_central_binom_of_binomial_carrywise_bound(
    n: Nat, k: Nat
) {
    k < n and falling_product_binomial_central_binom_carrywise_bound(n, k)
        implies falling_product(n, k).divides(central_binom(n))
} by {
    if k < n and falling_product_binomial_central_binom_carrywise_bound(n, k) {
        falling_product_central_binom_carrywise_bound_of_binomial_carrywise_bound(n, k)
        falling_product_central_binom_carrywise_bound(n, k)
        falling_product_divides_central_binom_of_carrywise_bound(n, k)
        falling_product(n, k).divides(central_binom(n))
    }
}

/// Falling-product divisibility of the central binomial coefficient implies
/// the binomial-Legendre central-binomial primewise bounds.
theorem falling_product_binomial_central_binom_primewise_bound_of_divides(
    n: Nat, k: Nat
) {
    k < n and falling_product(n, k).divides(central_binom(n))
        implies falling_product_binomial_central_binom_primewise_bound(n, k)
} by {
    if k < n and falling_product(n, k).divides(central_binom(n)) {
        falling_product_central_binom_primewise_bound_of_divides(n, k)
        falling_product_central_binom_primewise_bound(n, k)
        falling_product_binomial_central_binom_primewise_bound_of_primewise_bound(n, k)
        falling_product_binomial_central_binom_primewise_bound(n, k)
    }
}

/// Falling-product divisibility of the central binomial coefficient implies
/// the binomial-Legendre central-binomial carrywise bounds.
theorem falling_product_binomial_central_binom_carrywise_bound_of_divides(
    n: Nat, k: Nat
) {
    k < n and falling_product(n, k).divides(central_binom(n))
        implies falling_product_binomial_central_binom_carrywise_bound(n, k)
} by {
    if k < n and falling_product(n, k).divides(central_binom(n)) {
        falling_product_central_binom_carrywise_bound_of_divides(n, k)
        falling_product_central_binom_carrywise_bound(n, k)
        falling_product_binomial_central_binom_carrywise_bound_of_carrywise_bound(n, k)
        falling_product_binomial_central_binom_carrywise_bound(n, k)
    }
}
