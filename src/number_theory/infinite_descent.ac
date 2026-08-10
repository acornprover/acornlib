/// Infinite descent: the classical method of proving impossibility by
/// constructing smaller and smaller solutions.
///
/// This file gathers the descent arguments built on the classical pattern:
/// from a solution, construct a strictly smaller one, and rule the descent
/// out by well-founded induction.  The √2 descent of diophantine.ac
/// (Section 3 there) is included in full so that this file is self-contained
/// (Section 1); the remaining sections build on it:
///
///   1. The irrationality of √2: the Diophantine equation x² = 2y² has no
///      nonzero natural solution (mirrors diophantine.ac, Section 3).
///
///   2. The irrationality of √3: the Diophantine equation x² = 3y² has no
///      nonzero natural solution.  From a solution, three divides both
///      coordinates, so dividing by three yields a smaller solution;
///      well-founded induction on the first coordinate rules the descent out
///      (also proved in approximation_deep.ac).
///
///   3. The equation x² + y² = 3z² has no nonzero natural solution.  Modulo
///      three, x² + y² ≡ 0 forces x ≡ y ≡ 0 (squares are 0 or 1 modulo 3),
///      so a solution yields x = 3a, y = 3b, z = 3c with a² + b² = 3c², a
///      smaller solution; descent on the third coordinate rules it out.
///
///   4. √n is irrational for nonsquare n: the general statement is recorded,
///      and the base cases n = 2 and n = 3 are proved from the two descents.
///
///   5. Fermat's descent x⁴ + y⁴ = z² has no nonzero solution: stated but not
///      proved, since the classical descent for the fourth power needs the
///      Pythagorean-triple parameterization in primitive form plus the
///      two-squares descent step, which the library does not yet assemble.
///
///   6. The two-squares descent: a prime p ≡ 3 (mod 4) dividing x² + y²
///      divides both x and y.  The proof runs modulo p: if p ∤ y, the inverse
///      of y modulo p turns p | x² + y² into (x·y⁻¹)² ≡ -1 (mod p), so p - 1
///      is a quadratic residue modulo p; but by the first supplement to
///      quadratic reciprocity, p - 1 is a quadratic residue exactly when
///      p ≡ 1 (mod 4), contradicting p ≡ 3 (mod 4).
from nat import Nat, add_comm, add_assoc, mul_comm, mul_assoc, distrib_right,
    distrib_left, mul_to_zero, mul_cancel_left, gcd_of_prime, divides_mul,
    divides_sub, divides_self, lte_imp_not_lt, lt_or_lte, lte_trans, lt_trans,
    lt_add_left, lt_add_suc, divisor_lt, lte_ref, lte_mul_both, lte_add_left,
    lte_add_right, lt_and_lte, only_zero_lte_zero, lt_not_ref, lt_suc,
    lt_suc_right, not_lt_zero, lt_imp_lt_suc, small_mod, mod_of_zero,
    div_imp_mod, add_mod, alt_suc_ne_zero, add_zero_right, add_zero_left,
    sq_eq_mul, add_sub, add_imp_sub
from algebra.well_founded import nat_lt_relation, nat_lt_relation_induction_at
from order import lt_imp_lte
from number_theory.coprime import coprime_divides_of_divides_mul
from number_theory.congruence import mod_add_eq, mod_mul_eq, mod_add_mul, mod_lt,
    mod_congr_mod_self, congr_mod_symm, congr_mod_mul, congr_mod_zero_of_divides,
    divides_of_congr_mod_zero, congr_mod_zero_iff_divides, congr_mod_refl,
    congr_mod_trans, congr_mod_add, congr_mod_pow
from number_theory.goldbach import three_is_prime
from number_theory.factorisation import no_proper_divisor_imp_prime
from number_theory.quadratic_residue import is_quadratic_residue_mod
from number_theory.quadratic_residue_supplements import prime_pred_quadratic_residue_iff_congr_one_mod_four
from number_theory.totient import congr_mod_add_cancel_right_pos,
    not_coprime_imp_divides_prime
from number_theory.modular_inverse import nat_modular_inverse_exists_pos

numerals Nat

// ============================================================================
// Section 1: x² = 2y² has no nonzero solution (the irrationality of √2)
// ============================================================================

/// Two is prime.
theorem nat_two_prime_local {
    Nat.2.is_prime
} by {
    Nat.1 < Nat.2
    forall(k: Nat) {
        if Nat.1 < k and k < Nat.2 {
            false
        }
    }
    no_proper_divisor_imp_prime(Nat.2)
}

/// If two divides a square, it divides the root (two is prime).
theorem two_divides_square_imp_two_divides(x: Nat) {
    Nat.2.divides(x * x) implies Nat.2.divides(x)
} by {
    if Nat.2.divides(x * x) {
        nat_two_prime_local
        gcd_of_prime(Nat.2, x)
        if Nat.2.gcd(x) = Nat.1 {
            Nat.2.coprime(x)
            coprime_divides_of_divides_mul(Nat.2, x, x)
            Nat.2.divides(x)
        } else {
            Nat.2.divides(x)
        }
    }
}

/// The square of a doubled number: (2x)² = 4x².
theorem nat_sq_double(x: Nat) {
    (Nat.2 * x) * (Nat.2 * x) = Nat.4 * (x * x)
} by {
    (Nat.2 * x) * (Nat.2 * x) = Nat.2 * x * Nat.2 * x
    Nat.2 * x * Nat.2 * x = Nat.2 * Nat.2 * x * x
    Nat.2 * Nat.2 = Nat.4
    Nat.2 * Nat.2 * x * x = Nat.4 * (x * x)
    (Nat.2 * x) * (Nat.2 * x) = Nat.4 * (x * x)
}

/// Infinite descent: a nonzero solution of x² = 2y² yields a smaller one.
/// From x² = 2y² both coordinates are even, x = 2a and y = 2b, and then
/// a² = 2b² with a < x.
theorem sq_eq_two_sq_descent(x: Nat, y: Nat) {
    x != Nat.0 and x * x = Nat.2 * (y * y) implies
        exists(a: Nat, b: Nat) {
            a < x and Nat.2 * a = x and a * a = Nat.2 * (b * b)
        }
} by {
    if x != Nat.0 and x * x = Nat.2 * (y * y) {
        // 2 | x·x, hence 2 | x, x = 2a.
        Nat.2.divides(Nat.2 * (y * y))
        Nat.2 * (y * y) = x * x
        Nat.2.divides(x * x)
        two_divides_square_imp_two_divides(x)
        Nat.2.divides(x)
        let (a: Nat) satisfy { Nat.2 * a = x }
        // a is a proper smaller root: a < x.
        mul_to_zero(Nat.2, a)
        if Nat.2 * a != Nat.0 {
            a != Nat.0
        }
        a != Nat.0
        a * Nat.2 = Nat.2 * a
        a * Nat.2 = x
        Nat.1 < Nat.2
        divisor_lt(a, Nat.2, x)
        a < x
        // y·y = 2·a·a from (2a)² = 2y².
        Nat.2 * a = x
        x * x = (Nat.2 * a) * (Nat.2 * a)
        x * x = Nat.2 * (y * y)
        (Nat.2 * a) * (Nat.2 * a) = Nat.2 * (y * y)
        nat_sq_double(a)
        (Nat.2 * a) * (Nat.2 * a) = Nat.4 * (a * a)
        Nat.4 * (a * a) = Nat.2 * (y * y)
        Nat.2 * (Nat.2 * (a * a)) = Nat.2 * (y * y)
        mul_cancel_left(Nat.2, Nat.2 * (a * a), y * y)
        Nat.2 * (a * a) = y * y
        // 2 | y·y, hence 2 | y, y = 2b.
        Nat.2.divides(Nat.2 * (a * a))
        Nat.2 * (a * a) = y * y
        Nat.2.divides(y * y)
        two_divides_square_imp_two_divides(y)
        Nat.2.divides(y)
        let (b: Nat) satisfy { Nat.2 * b = y }
        // a·a = 2·b·b from y = 2b.
        Nat.2 * b = y
        y * y = (Nat.2 * b) * (Nat.2 * b)
        Nat.2 * (a * a) = y * y
        Nat.2 * (a * a) = (Nat.2 * b) * (Nat.2 * b)
        nat_sq_double(b)
        (Nat.2 * b) * (Nat.2 * b) = Nat.4 * (b * b)
        Nat.2 * (a * a) = Nat.4 * (b * b)
        Nat.2 * (a * a) = Nat.2 * (Nat.2 * (b * b))
        mul_cancel_left(Nat.2, a * a, Nat.2 * (b * b))
        a * a = Nat.2 * (b * b)
        // assemble the witness.
        a < x and Nat.2 * a = x and a * a = Nat.2 * (b * b)
        exists(a2: Nat, b2: Nat) {
            a2 < x and Nat.2 * a2 = x and a2 * a2 = Nat.2 * (b2 * b2)
        }
    }
}

/// The infinite descent of `sq_eq_two_sq_descent`, run by well-founded
/// induction on the first coordinate: x² = 2y² forces x = 0.
theorem sq_eq_two_sq_zero(x: Nat) {
    forall(y: Nat) { x * x = Nat.2 * (y * y) implies x = Nat.0 }
} by {
    let f: Nat -> Bool = function(t: Nat) {
        forall(y: Nat) { t * t = Nat.2 * (y * y) implies t = Nat.0 }
    }
    forall(z: Nat) {
        if forall(k: Nat) { nat_lt_relation(k, z) implies f(k) } {
            forall(y: Nat) {
                if z * z = Nat.2 * (y * y) {
                    if z = Nat.0 {
                        z = Nat.0
                    } else {
                        z != Nat.0 and z * z = Nat.2 * (y * y)
                        sq_eq_two_sq_descent(z, y)
                        let (a: Nat, b: Nat) satisfy {
                            a < z and Nat.2 * a = z and a * a = Nat.2 * (b * b)
                        }
                        nat_lt_relation(a, z) = (a < z)
                        nat_lt_relation(a, z)
                        forall(k: Nat) { nat_lt_relation(k, z) implies f(k) }
                        nat_lt_relation(a, z) implies f(a)
                        f(a)
                        f(a) =
                            forall(w: Nat) { a * a = Nat.2 * (w * w) implies a = Nat.0 }
                        forall(w: Nat) { a * a = Nat.2 * (w * w) implies a = Nat.0 }
                        a * a = Nat.2 * (b * b) implies a = Nat.0
                        a = Nat.0
                        Nat.2 * a = z
                        Nat.2 * Nat.0 = Nat.0
                        z = Nat.0
                        false
                    }
                }
            }
            forall(y: Nat) { z * z = Nat.2 * (y * y) implies z = Nat.0 }
            f(z) = forall(w: Nat) { z * z = Nat.2 * (w * w) implies z = Nat.0 }
            f(z)
        }
    }
    nat_lt_relation_induction_at(f, x)
    f(x)
    f(x) = forall(y: Nat) { x * x = Nat.2 * (y * y) implies x = Nat.0 }
    forall(y: Nat) { x * x = Nat.2 * (y * y) implies x = Nat.0 }
}

/// The Diophantine equation x² = 2y² has no nonzero natural solution.
theorem no_nontrivial_sq_eq_two_sq(x: Nat, y: Nat) {
    x * x = Nat.2 * (y * y) implies x = Nat.0 and y = Nat.0
} by {
    if x * x = Nat.2 * (y * y) {
        sq_eq_two_sq_zero(x)
        forall(w: Nat) { x * x = Nat.2 * (w * w) implies x = Nat.0 }
        x * x = Nat.2 * (y * y) implies x = Nat.0
        x = Nat.0
        x * x = Nat.2 * (y * y)
        Nat.0 * Nat.0 = Nat.0
        Nat.0 = Nat.2 * (y * y)
        Nat.2 * (y * y) = Nat.0
        mul_to_zero(Nat.2, y * y)
        if Nat.2 = Nat.0 {
            false
        }
        y * y = Nat.0
        mul_to_zero(y, y)
        if y = Nat.0 {
        } else {
            false
        }
        y = Nat.0
        x = Nat.0 and y = Nat.0
    }
}

// ============================================================================
// Section 2: x² = 3y² has no nonzero solution (the irrationality of √3)
// ============================================================================

/// If three divides a square, it divides the root (three is prime).
theorem three_divides_square_imp_three_divides(x: Nat) {
    Nat.3.divides(x * x) implies Nat.3.divides(x)
} by {
    if Nat.3.divides(x * x) {
        three_is_prime
        gcd_of_prime(Nat.3, x)
        if Nat.3.gcd(x) = Nat.1 {
            Nat.3.coprime(x)
            coprime_divides_of_divides_mul(Nat.3, x, x)
            Nat.3.divides(x)
        } else {
            Nat.3.divides(x)
        }
    }
}

/// The square of a tripled number: (3x)² = 9x².
theorem nat_sq_triple(x: Nat) {
    (Nat.3 * x) * (Nat.3 * x) = Nat.9 * (x * x)
} by {
    (Nat.3 * x) * (Nat.3 * x) = Nat.3 * x * Nat.3 * x
    Nat.3 * x * Nat.3 * x = Nat.3 * Nat.3 * x * x
    Nat.3 * Nat.3 = Nat.9
    Nat.3 * Nat.3 * x * x = Nat.9 * (x * x)
    (Nat.3 * x) * (Nat.3 * x) = Nat.9 * (x * x)
}

/// Infinite descent: a nonzero solution of x² = 3y² yields a smaller one.
/// From x² = 3y² both coordinates are divisible by three, x = 3a and
/// y = 3b, and then a² = 3b² with a < x.
theorem sq_eq_three_sq_descent(x: Nat, y: Nat) {
    x != Nat.0 and x * x = Nat.3 * (y * y) implies
        exists(a: Nat, b: Nat) {
            a < x and Nat.3 * a = x and a * a = Nat.3 * (b * b)
        }
} by {
    if x != Nat.0 and x * x = Nat.3 * (y * y) {
        // 3 | x·x, hence 3 | x, x = 3a.
        Nat.3.divides(Nat.3 * (y * y))
        Nat.3 * (y * y) = x * x
        Nat.3.divides(x * x)
        three_divides_square_imp_three_divides(x)
        Nat.3.divides(x)
        let (a: Nat) satisfy { Nat.3 * a = x }
        // a is a proper smaller root: a < x.
        mul_to_zero(Nat.3, a)
        if Nat.3 * a != Nat.0 {
            a != Nat.0
        }
        a != Nat.0
        a * Nat.3 = Nat.3 * a
        a * Nat.3 = x
        Nat.1 < Nat.3
        divisor_lt(a, Nat.3, x)
        a < x
        // y·y = 3·a·a from (3a)² = 3y².
        Nat.3 * a = x
        x * x = (Nat.3 * a) * (Nat.3 * a)
        x * x = Nat.3 * (y * y)
        (Nat.3 * a) * (Nat.3 * a) = Nat.3 * (y * y)
        nat_sq_triple(a)
        (Nat.3 * a) * (Nat.3 * a) = Nat.9 * (a * a)
        Nat.9 * (a * a) = Nat.3 * (y * y)
        Nat.3 * (Nat.3 * (a * a)) = Nat.3 * (y * y)
        mul_cancel_left(Nat.3, Nat.3 * (a * a), y * y)
        Nat.3 * (a * a) = y * y
        // 3 | y·y, hence 3 | y, y = 3b.
        Nat.3.divides(Nat.3 * (a * a))
        Nat.3 * (a * a) = y * y
        Nat.3.divides(y * y)
        three_divides_square_imp_three_divides(y)
        Nat.3.divides(y)
        let (b: Nat) satisfy { Nat.3 * b = y }
        // a·a = 3·b·b from y = 3b.
        Nat.3 * b = y
        y * y = (Nat.3 * b) * (Nat.3 * b)
        Nat.3 * (a * a) = y * y
        Nat.3 * (a * a) = (Nat.3 * b) * (Nat.3 * b)
        nat_sq_triple(b)
        (Nat.3 * b) * (Nat.3 * b) = Nat.9 * (b * b)
        Nat.3 * (a * a) = Nat.9 * (b * b)
        Nat.3 * (a * a) = Nat.3 * (Nat.3 * (b * b))
        mul_cancel_left(Nat.3, a * a, Nat.3 * (b * b))
        a * a = Nat.3 * (b * b)
        // assemble the witness.
        a < x and Nat.3 * a = x and a * a = Nat.3 * (b * b)
        exists(a2: Nat, b2: Nat) {
            a2 < x and Nat.3 * a2 = x and a2 * a2 = Nat.3 * (b2 * b2)
        }
    }
}

/// The infinite descent of `sq_eq_three_sq_descent`, run by well-founded
/// induction on the first coordinate: x² = 3y² forces x = 0.
theorem sq_eq_three_sq_zero(x: Nat) {
    forall(y: Nat) { x * x = Nat.3 * (y * y) implies x = Nat.0 }
} by {
    let f: Nat -> Bool = function(t: Nat) {
        forall(y: Nat) { t * t = Nat.3 * (y * y) implies t = Nat.0 }
    }
    forall(z: Nat) {
        if forall(k: Nat) { nat_lt_relation(k, z) implies f(k) } {
            forall(y: Nat) {
                if z * z = Nat.3 * (y * y) {
                    if z = Nat.0 {
                        z = Nat.0
                    } else {
                        z != Nat.0 and z * z = Nat.3 * (y * y)
                        sq_eq_three_sq_descent(z, y)
                        let (a: Nat, b: Nat) satisfy {
                            a < z and Nat.3 * a = z and a * a = Nat.3 * (b * b)
                        }
                        nat_lt_relation(a, z) = (a < z)
                        nat_lt_relation(a, z)
                        forall(k: Nat) { nat_lt_relation(k, z) implies f(k) }
                        nat_lt_relation(a, z) implies f(a)
                        f(a)
                        f(a) =
                            forall(w: Nat) { a * a = Nat.3 * (w * w) implies a = Nat.0 }
                        forall(w: Nat) { a * a = Nat.3 * (w * w) implies a = Nat.0 }
                        a * a = Nat.3 * (b * b) implies a = Nat.0
                        a = Nat.0
                        Nat.3 * a = z
                        Nat.3 * Nat.0 = Nat.0
                        z = Nat.0
                        false
                    }
                }
            }
            forall(y: Nat) { z * z = Nat.3 * (y * y) implies z = Nat.0 }
            f(z) = forall(w: Nat) { z * z = Nat.3 * (w * w) implies z = Nat.0 }
            f(z)
        }
    }
    nat_lt_relation_induction_at(f, x)
    f(x)
    f(x) = forall(y: Nat) { x * x = Nat.3 * (y * y) implies x = Nat.0 }
    forall(y: Nat) { x * x = Nat.3 * (y * y) implies x = Nat.0 }
}

/// The Diophantine equation x² = 3y² has no nonzero natural solution.
theorem no_nontrivial_sq_eq_three_sq(x: Nat, y: Nat) {
    x * x = Nat.3 * (y * y) implies x = Nat.0 and y = Nat.0
} by {
    if x * x = Nat.3 * (y * y) {
        sq_eq_three_sq_zero(x)
        forall(w: Nat) { x * x = Nat.3 * (w * w) implies x = Nat.0 }
        x * x = Nat.3 * (y * y) implies x = Nat.0
        x = Nat.0
        x * x = Nat.3 * (y * y)
        Nat.0 * Nat.0 = Nat.0
        Nat.0 = Nat.3 * (y * y)
        Nat.3 * (y * y) = Nat.0
        mul_to_zero(Nat.3, y * y)
        if Nat.3 = Nat.0 {
            false
        }
        y * y = Nat.0
        mul_to_zero(y, y)
        if y = Nat.0 {
        } else {
            false
        }
        y = Nat.0
        x = Nat.0 and y = Nat.0
    }
}

// ============================================================================
// Section 3: x² + y² = 3z² has no nonzero solution
// ============================================================================

/// One modulo three is one.
theorem one_mod_three {
    Nat.1.mod(Nat.3) = Nat.1
} by {
    lt_suc(Nat.1)
    Nat.1 < Nat.2
    lt_imp_lt_suc(Nat.1, Nat.2)
    Nat.1 < Nat.3
    small_mod(Nat.1, Nat.3)
    Nat.1.mod(Nat.3) = Nat.1
}

/// Two modulo three is two.
theorem two_mod_three {
    Nat.2.mod(Nat.3) = Nat.2
} by {
    lt_suc(Nat.2)
    Nat.2 < Nat.3
    small_mod(Nat.2, Nat.3)
    Nat.2.mod(Nat.3) = Nat.2
}

/// The square of a natural number is congruent to zero or one modulo three.
theorem square_mod_three_case(a: Nat) {
    (a * a).mod(Nat.3) = Nat.0 or (a * a).mod(Nat.3) = Nat.1
} by {
    let r: Nat = a.mod(Nat.3)
    add_mod(a, Nat.3)
    let q: Nat satisfy { q * Nat.3 + a.mod(Nat.3) = a }
    q * Nat.3 + r = a
    mod_congr_mod_self(a, Nat.3)
    a.mod(Nat.3).congr_mod(a, Nat.3)
    congr_mod_symm(a.mod(Nat.3), a, Nat.3)
    a.congr_mod(a.mod(Nat.3), Nat.3)
    a.congr_mod(r, Nat.3)
    congr_mod_mul(a, r, a, r, Nat.3)
    (a * a).congr_mod(r * r, Nat.3)
    (a * a).mod(Nat.3) = (r * r).mod(Nat.3)
    alt_suc_ne_zero(Nat.2)
    Nat.3 != Nat.0
    mod_lt(a, Nat.3)
    r < Nat.3
    lt_suc_right(r, Nat.2)
    if r != Nat.2 {
        r < Nat.2
        lt_suc_right(r, Nat.1)
        if r != Nat.1 {
            r < Nat.1
            lt_suc_right(r, Nat.0)
            if r != Nat.0 {
                r < Nat.0
                not_lt_zero(r)
                false
            }
            r = Nat.0
            r * r = Nat.0
            mod_of_zero(Nat.3)
            Nat.0.mod(Nat.3) = Nat.0
            (r * r).mod(Nat.3) = Nat.0
            (a * a).mod(Nat.3) = Nat.0
            (a * a).mod(Nat.3) = Nat.0 or (a * a).mod(Nat.3) = Nat.1
        } else {
            r = Nat.1
            r * r = Nat.1
            one_mod_three
            Nat.1.mod(Nat.3) = Nat.1
            (r * r).mod(Nat.3) = Nat.1
            (a * a).mod(Nat.3) = Nat.1
            (a * a).mod(Nat.3) = Nat.0 or (a * a).mod(Nat.3) = Nat.1
        }
    } else {
        r = Nat.2
        r * r = Nat.4
        Nat.4 = Nat.1 * Nat.3 + Nat.1
        mod_add_mul(Nat.1, Nat.3, Nat.1)
        (Nat.1 * Nat.3 + Nat.1).mod(Nat.3) = Nat.1.mod(Nat.3)
        one_mod_three
        Nat.1.mod(Nat.3) = Nat.1
        Nat.4.mod(Nat.3) = Nat.1
        (r * r).mod(Nat.3) = Nat.1
        (a * a).mod(Nat.3) = Nat.1
        (a * a).mod(Nat.3) = Nat.0 or (a * a).mod(Nat.3) = Nat.1
    }
}

/// If a sum of two naturals from {0, 1} vanishes modulo 3, the first summand
/// is zero: the only pair from {0, 1} summing to a multiple of three is
/// 0 + 0.
theorem mod_three_sum_pair_zero_imp_u_zero(u: Nat, v: Nat) {
    (u + v).mod(Nat.3) = Nat.0 and (u = Nat.0 or u = Nat.1) and (v = Nat.0 or v = Nat.1)
        implies u = Nat.0
} by {
    if (u + v).mod(Nat.3) = Nat.0 and (u = Nat.0 or u = Nat.1) and (v = Nat.0 or v = Nat.1) {
        (u + v).mod(Nat.3) = Nat.0
        if u = Nat.0 {
            u = Nat.0
        } else {
            u = Nat.1
            if v = Nat.0 {
                u + v = Nat.1
                (u + v).mod(Nat.3) = Nat.1.mod(Nat.3)
                (u + v).mod(Nat.3) = Nat.0
                Nat.1.mod(Nat.3) = Nat.0
                one_mod_three
                Nat.1.mod(Nat.3) = Nat.1
                Nat.1 = Nat.0
                false
            } else {
                v = Nat.1
                u + v = Nat.2
                (u + v).mod(Nat.3) = Nat.2.mod(Nat.3)
                (u + v).mod(Nat.3) = Nat.0
                Nat.2.mod(Nat.3) = Nat.0
                two_mod_three
                Nat.2.mod(Nat.3) = Nat.2
                Nat.2 = Nat.0
                false
            }
        }
    }
}

/// If a sum of two naturals from {0, 1} vanishes modulo 3, the second
/// summand is zero.
theorem mod_three_sum_pair_zero_imp_v_zero(u: Nat, v: Nat) {
    (u + v).mod(Nat.3) = Nat.0 and (u = Nat.0 or u = Nat.1) and (v = Nat.0 or v = Nat.1)
        implies v = Nat.0
} by {
    if (u + v).mod(Nat.3) = Nat.0 and (u = Nat.0 or u = Nat.1) and (v = Nat.0 or v = Nat.1) {
        (u + v).mod(Nat.3) = Nat.0
        if v = Nat.0 {
            v = Nat.0
        } else {
            v = Nat.1
            if u = Nat.0 {
                u + v = Nat.1
                (u + v).mod(Nat.3) = Nat.1.mod(Nat.3)
                (u + v).mod(Nat.3) = Nat.0
                Nat.1.mod(Nat.3) = Nat.0
                one_mod_three
                Nat.1.mod(Nat.3) = Nat.1
                Nat.1 = Nat.0
                false
            } else {
                u = Nat.1
                u + v = Nat.2
                (u + v).mod(Nat.3) = Nat.2.mod(Nat.3)
                (u + v).mod(Nat.3) = Nat.0
                Nat.2.mod(Nat.3) = Nat.0
                two_mod_three
                Nat.2.mod(Nat.3) = Nat.2
                Nat.2 = Nat.0
                false
            }
        }
    }
}

/// If three divides a sum of two squares, it divides both summand roots:
/// squares are 0 or 1 modulo 3, and only 0 + 0 sums to 0 modulo 3.
theorem three_divides_sum_two_squares(x: Nat, y: Nat) {
    Nat.3.divides(x * x + y * y) implies Nat.3.divides(x) and Nat.3.divides(y)
} by {
    if Nat.3.divides(x * x + y * y) {
        congr_mod_zero_of_divides(Nat.3, x * x + y * y)
        (x * x + y * y).congr_mod(Nat.0, Nat.3)
        (x * x + y * y).mod(Nat.3) = Nat.0.mod(Nat.3)
        mod_of_zero(Nat.3)
        Nat.0.mod(Nat.3) = Nat.0
        (x * x + y * y).mod(Nat.3) = Nat.0
        mod_add_eq(x * x, y * y, Nat.3)
        (x * x + y * y).mod(Nat.3) = ((x * x).mod(Nat.3) + (y * y).mod(Nat.3)).mod(Nat.3)
        ((x * x).mod(Nat.3) + (y * y).mod(Nat.3)).mod(Nat.3) = Nat.0
        square_mod_three_case(x)
        (x * x).mod(Nat.3) = Nat.0 or (x * x).mod(Nat.3) = Nat.1
        square_mod_three_case(y)
        (y * y).mod(Nat.3) = Nat.0 or (y * y).mod(Nat.3) = Nat.1
        mod_three_sum_pair_zero_imp_u_zero((x * x).mod(Nat.3), (y * y).mod(Nat.3))
        (x * x).mod(Nat.3) = Nat.0
        mod_three_sum_pair_zero_imp_v_zero((x * x).mod(Nat.3), (y * y).mod(Nat.3))
        (y * y).mod(Nat.3) = Nat.0
        (x * x).congr_mod(Nat.0, Nat.3)
        divides_of_congr_mod_zero(Nat.3, x * x)
        Nat.3.divides(x * x)
        three_divides_square_imp_three_divides(x)
        Nat.3.divides(x)
        (y * y).congr_mod(Nat.0, Nat.3)
        divides_of_congr_mod_zero(Nat.3, y * y)
        Nat.3.divides(y * y)
        three_divides_square_imp_three_divides(y)
        Nat.3.divides(y)
        Nat.3.divides(x) and Nat.3.divides(y)
    }
}

/// Infinite descent: a nonzero solution of x² + y² = 3z² yields a smaller
/// one.  Modulo 3, x² + y² ≡ 0 forces x ≡ y ≡ 0 (mod 3), so x = 3a and
/// y = 3b; then a² + b² = 3c² with c < z.
theorem sq_sum_three_sq_descent(x: Nat, y: Nat, z: Nat) {
    z != Nat.0 and x * x + y * y = Nat.3 * (z * z) implies
        exists(a: Nat, b: Nat, c: Nat) {
            c < z and Nat.3 * c = z and a * a + b * b = Nat.3 * (c * c)
        }
} by {
    if z != Nat.0 and x * x + y * y = Nat.3 * (z * z) {
        // 3 | x·x + y·y, hence 3 | x and 3 | y, x = 3a, y = 3b.
        Nat.3.divides(Nat.3 * (z * z))
        Nat.3 * (z * z) = x * x + y * y
        Nat.3.divides(x * x + y * y)
        three_divides_sum_two_squares(x, y)
        Nat.3.divides(x) and Nat.3.divides(y)
        Nat.3.divides(x)
        let (a: Nat) satisfy { Nat.3 * a = x }
        Nat.3.divides(y)
        let (b: Nat) satisfy { Nat.3 * b = y }
        // 3·(a·a + b·b) = z·z from (3a)² + (3b)² = 3z².
        Nat.3 * a = x
        Nat.3 * b = y
        x * x = (Nat.3 * a) * (Nat.3 * a)
        y * y = (Nat.3 * b) * (Nat.3 * b)
        x * x + y * y = (Nat.3 * a) * (Nat.3 * a) + (Nat.3 * b) * (Nat.3 * b)
        x * x + y * y = Nat.3 * (z * z)
        (Nat.3 * a) * (Nat.3 * a) + (Nat.3 * b) * (Nat.3 * b) = Nat.3 * (z * z)
        nat_sq_triple(a)
        (Nat.3 * a) * (Nat.3 * a) = Nat.9 * (a * a)
        nat_sq_triple(b)
        (Nat.3 * b) * (Nat.3 * b) = Nat.9 * (b * b)
        Nat.9 * (a * a) + Nat.9 * (b * b) = Nat.3 * (z * z)
        distrib_right(Nat.9, a * a, b * b)
        Nat.9 * (a * a + b * b) = Nat.9 * (a * a) + Nat.9 * (b * b)
        Nat.9 * (a * a + b * b) = Nat.3 * (z * z)
        Nat.9 = Nat.3 * Nat.3
        Nat.3 * (Nat.3 * (a * a + b * b)) = Nat.3 * (z * z)
        mul_cancel_left(Nat.3, Nat.3 * (a * a + b * b), z * z)
        Nat.3 * (a * a + b * b) = z * z
        // 3 | z·z, hence 3 | z, z = 3c.
        Nat.3.divides(Nat.3 * (a * a + b * b))
        Nat.3 * (a * a + b * b) = z * z
        Nat.3.divides(z * z)
        three_divides_square_imp_three_divides(z)
        Nat.3.divides(z)
        let (c: Nat) satisfy { Nat.3 * c = z }
        // a·a + b·b = 3·c·c from z = 3c.
        Nat.3 * c = z
        z * z = (Nat.3 * c) * (Nat.3 * c)
        Nat.3 * (a * a + b * b) = z * z
        Nat.3 * (a * a + b * b) = (Nat.3 * c) * (Nat.3 * c)
        nat_sq_triple(c)
        (Nat.3 * c) * (Nat.3 * c) = Nat.9 * (c * c)
        Nat.3 * (a * a + b * b) = Nat.9 * (c * c)
        Nat.9 * (c * c) = Nat.3 * (Nat.3 * (c * c))
        Nat.3 * (a * a + b * b) = Nat.3 * (Nat.3 * (c * c))
        mul_cancel_left(Nat.3, a * a + b * b, Nat.3 * (c * c))
        a * a + b * b = Nat.3 * (c * c)
        // c is a proper smaller root: c < z.
        mul_to_zero(Nat.3, c)
        if Nat.3 * c != Nat.0 {
            c != Nat.0
        }
        c != Nat.0
        c * Nat.3 = Nat.3 * c
        c * Nat.3 = z
        Nat.1 < Nat.3
        divisor_lt(c, Nat.3, z)
        c < z
        // assemble the witness.
        c < z and Nat.3 * c = z and a * a + b * b = Nat.3 * (c * c)
        exists(a2: Nat, b2: Nat, c2: Nat) {
            c2 < z and Nat.3 * c2 = z and a2 * a2 + b2 * b2 = Nat.3 * (c2 * c2)
        }
    }
}

/// The infinite descent of `sq_sum_three_sq_descent`, run by well-founded
/// induction on the third coordinate: x² + y² = 3z² forces z = 0.
theorem sq_sum_three_sq_zero(z: Nat) {
    forall(x: Nat, y: Nat) { x * x + y * y = Nat.3 * (z * z) implies z = Nat.0 }
} by {
    let f: Nat -> Bool = function(t: Nat) {
        forall(x: Nat, y: Nat) { x * x + y * y = Nat.3 * (t * t) implies t = Nat.0 }
    }
    forall(w: Nat) {
        if forall(k: Nat) { nat_lt_relation(k, w) implies f(k) } {
            forall(x: Nat, y: Nat) {
                if x * x + y * y = Nat.3 * (w * w) {
                    if w = Nat.0 {
                        w = Nat.0
                    } else {
                        w != Nat.0 and x * x + y * y = Nat.3 * (w * w)
                        sq_sum_three_sq_descent(x, y, w)
                        let (a: Nat, b: Nat, c: Nat) satisfy {
                            c < w and Nat.3 * c = w and a * a + b * b = Nat.3 * (c * c)
                        }
                        nat_lt_relation(c, w) = (c < w)
                        nat_lt_relation(c, w)
                        forall(k: Nat) { nat_lt_relation(k, w) implies f(k) }
                        nat_lt_relation(c, w) implies f(c)
                        f(c)
                        f(c) =
                            forall(u: Nat, v: Nat) {
                                u * u + v * v = Nat.3 * (c * c) implies c = Nat.0
                            }
                        forall(u: Nat, v: Nat) {
                            u * u + v * v = Nat.3 * (c * c) implies c = Nat.0
                        }
                        a * a + b * b = Nat.3 * (c * c) implies c = Nat.0
                        c = Nat.0
                        Nat.3 * c = w
                        Nat.3 * Nat.0 = Nat.0
                        w = Nat.0
                        false
                    }
                }
            }
            forall(x: Nat, y: Nat) { x * x + y * y = Nat.3 * (w * w) implies w = Nat.0 }
            f(w) = forall(u: Nat, v: Nat) {
                u * u + v * v = Nat.3 * (w * w) implies w = Nat.0
            }
            f(w)
        }
    }
    nat_lt_relation_induction_at(f, z)
    f(z)
    f(z) = forall(x: Nat, y: Nat) { x * x + y * y = Nat.3 * (z * z) implies z = Nat.0 }
    forall(x: Nat, y: Nat) { x * x + y * y = Nat.3 * (z * z) implies z = Nat.0 }
}

/// The Diophantine equation x² + y² = 3z² has no nonzero natural solution:
/// the descent above forces z = 0, and then both squares vanish.
theorem no_nontrivial_sq_sum_three_sq(x: Nat, y: Nat, z: Nat) {
    x * x + y * y = Nat.3 * (z * z) implies x = Nat.0 and y = Nat.0 and z = Nat.0
} by {
    if x * x + y * y = Nat.3 * (z * z) {
        sq_sum_three_sq_zero(z)
        forall(u: Nat, v: Nat) {
            u * u + v * v = Nat.3 * (z * z) implies z = Nat.0
        }
        x * x + y * y = Nat.3 * (z * z) implies z = Nat.0
        z = Nat.0
        // the equation now reads x² + y² = 0.
        x * x + y * y = Nat.3 * (z * z)
        z * z = Nat.0 * Nat.0
        Nat.0 * Nat.0 = Nat.0
        Nat.3 * (z * z) = Nat.3 * Nat.0
        Nat.3 * Nat.0 = Nat.0
        x * x + y * y = Nat.0
        // x² <= x² + y², hence x² = 0, x = 0.
        only_zero_lte_zero(x * x)
        lte_add_left(x * x, Nat.0, y * y)
        Nat.0 <= y * y
        x * x + Nat.0 <= x * x + y * y
        x * x + Nat.0 = x * x
        x * x <= x * x + y * y
        x * x + y * y = Nat.0
        x * x <= Nat.0
        x * x = Nat.0
        mul_to_zero(x, x)
        if x = Nat.0 {
        } else {
            false
        }
        x = Nat.0
        // y² = 0 from x² + y² = 0 with x² = 0.
        x * x = Nat.0
        x * x + y * y = Nat.0
        Nat.0 + y * y = y * y
        y * y = Nat.0
        mul_to_zero(y, y)
        if y = Nat.0 {
        } else {
            false
        }
        y = Nat.0
        x = Nat.0 and y = Nat.0 and z = Nat.0
    }
}

// The equation x² + y² = 3z² has no nonzero integer solution either.  Taking
// absolute values gives |x|² + |y|² = 3·|z|², which reduces the integer
// statement to the natural one above; the reduction needs the library's
// absolute-value lemmas for a sum of squares, which are not yet assembled
// (the single-square version is `no_nontrivial_int_sq_eq_two_sq` in
// diophantine.ac).
//
// theorem no_nontrivial_int_sq_sum_three_sq(x: Int, y: Int, z: Int) {
//     x * x + y * y = Int.3 * (z * z) implies x = Int.0 and y = Int.0 and z = Int.0
// }

// ============================================================================
// Section 4: √n is irrational for nonsquare n
// ============================================================================

/// The Diophantine equation p² = 2q² has no solution with q ≠ 0: √2 is
/// irrational.  This is the descent of diophantine.ac restated in the
/// rational-denominator form.
theorem sqrt_two_irrational(p: Nat, q: Nat) {
    p * p = Nat.2 * (q * q) implies q = Nat.0
} by {
    if p * p = Nat.2 * (q * q) {
        no_nontrivial_sq_eq_two_sq(p, q)
        p = Nat.0 and q = Nat.0
        q = Nat.0
    }
}

/// The Diophantine equation p² = 3q² has no solution with q ≠ 0: √3 is
/// irrational.  This is the descent of Section 1 restated in the
/// rational-denominator form.
theorem sqrt_three_irrational(p: Nat, q: Nat) {
    p * p = Nat.3 * (q * q) implies q = Nat.0
} by {
    if p * p = Nat.3 * (q * q) {
        no_nontrivial_sq_eq_three_sq(p, q)
        p = Nat.0 and q = Nat.0
        q = Nat.0
    }
}

// The general statement — √n is irrational unless n is a perfect square.  In
// the Diophantine encoding (p/q)² = n with q ≠ 0 it reads:
//
// theorem sqrt_n_irrational_unless_square(n: Nat) {
//     not exists(m: Nat) { n = m * m } implies forall(p: Nat, q: Nat) {
//         p * p = n * (q * q) implies q = Nat.0
//     }
// }
//
// The base cases n = 2 and n = 3 are proved above (√2 by the descent in
// diophantine.ac, √3 by the descent in Section 1).  The general case would
// descend on a prime dividing n to an odd power; the library does not yet
// assemble the prime-factorization form of "not a perfect square" (see the
// comment at the end of approximation_deep.ac).

// ============================================================================
// Section 5: Fermat's descent x⁴ + y⁴ = z² (statement)
// ============================================================================

// Fermat's descent: the equation x⁴ + y⁴ = z² has no solution in nonzero
// natural numbers.  This is the classical proof that no right triangle has
// square area, the n = 4 case of Fermat's Last Theorem.  The standard descent
// starts from x⁴ + y⁴ = z², reuses the Pythagorean-triple parameterization
// (diophantine.ac, Section 1) in primitive form on the Pythagorean triple
// (x², y², z), and combines it with the two-squares descent step of
// Section 6 to produce a strictly smaller solution; the library does not yet
// assemble that combination, so the statement is recorded here without proof.
//
// theorem fermat_descent_fourth_power {
//     forall(x: Nat, y: Nat, z: Nat) {
//         x != Nat.0 and y != Nat.0 implies not (x.pow(Nat.4) + y.pow(Nat.4) = z * z)
//     }
// }

// ============================================================================
// Section 6: the two-squares descent
// ============================================================================

/// One modulo four is one.
theorem one_mod_four {
    Nat.1.mod(Nat.4) = Nat.1
} by {
    lt_suc(Nat.1)
    Nat.1 < Nat.2
    lt_imp_lt_suc(Nat.1, Nat.2)
    Nat.1 < Nat.3
    lt_imp_lt_suc(Nat.1, Nat.3)
    Nat.1 < Nat.4
    small_mod(Nat.1, Nat.4)
    Nat.1.mod(Nat.4) = Nat.1
}

/// Three modulo four is three.
theorem three_mod_four {
    Nat.3.mod(Nat.4) = Nat.3
} by {
    lt_suc(Nat.3)
    Nat.3 < Nat.4
    small_mod(Nat.3, Nat.4)
    Nat.3.mod(Nat.4) = Nat.3
}

/// A number congruent to three modulo four has the form 2(2q + 1) + 1: the
/// quotient q of the division by four gives p = 4q + 3 = 2(2q + 1) + 1.
theorem congr_three_mod_four_imp_half_decomp(p: Nat) {
    p.congr_mod(Nat.3, Nat.4) implies
        exists(q: Nat) { p = Nat.2 * (Nat.2 * q + Nat.1) + Nat.1 }
} by {
    if p.congr_mod(Nat.3, Nat.4) {
        p.mod(Nat.4) = Nat.3.mod(Nat.4)
        three_mod_four
        Nat.3.mod(Nat.4) = Nat.3
        p.mod(Nat.4) = Nat.3
        add_mod(p, Nat.4)
        let q: Nat satisfy { q * Nat.4 + p.mod(Nat.4) = p }
        q * Nat.4 + Nat.3 = p
        q * Nat.4 = Nat.4 * q
        Nat.4 * q + Nat.3 = p
        Nat.4 = Nat.2 * Nat.2
        (Nat.2 * Nat.2) * q + Nat.3 = p
        Nat.2 * (Nat.2 * q) + Nat.3 = p
        Nat.3 = Nat.2 + Nat.1
        Nat.2 * (Nat.2 * q) + (Nat.2 + Nat.1) = p
        Nat.2 * (Nat.2 * q) + Nat.2 = Nat.2 * (Nat.2 * q + Nat.1)
        Nat.2 * (Nat.2 * q + Nat.1) + Nat.1 = p
        p = Nat.2 * (Nat.2 * q + Nat.1) + Nat.1
        exists(q2: Nat) { p = Nat.2 * (Nat.2 * q2 + Nat.1) + Nat.1 }
    }
}

/// A number congruent to three modulo four is not congruent to one modulo
/// four.
theorem congr_three_mod_four_imp_not_congr_one_mod_four(p: Nat) {
    p.congr_mod(Nat.3, Nat.4) implies not p.congr_mod(Nat.1, Nat.4)
} by {
    if p.congr_mod(Nat.3, Nat.4) {
        p.mod(Nat.4) = Nat.3.mod(Nat.4)
        three_mod_four
        Nat.3.mod(Nat.4) = Nat.3
        p.mod(Nat.4) = Nat.3
        one_mod_four
        Nat.1.mod(Nat.4) = Nat.1
        Nat.3 != Nat.1
        p.mod(Nat.4) != Nat.1.mod(Nat.4)
        not p.congr_mod(Nat.1, Nat.4)
    }
}

/// The modular inverse step of the two-squares descent: if p divides x² + y²
/// and b is the inverse of y modulo p, then (x·b)² ≡ p - 1 (mod p).
///
/// From p | x² + y² we get (x² + y²)·b² ≡ 0 (mod p); expanding gives
/// x²·b² + y²·b² ≡ 0, and (y·b)² ≡ 1 replaces y²·b² by one, so
/// x²·b² + 1 ≡ 0 ≡ (p - 1) + 1 (mod p); cancelling the added one leaves
/// x²·b² ≡ p - 1 (mod p).
theorem two_squares_descent_inverse_step(p: Nat, x: Nat, y: Nat, b: Nat) {
    p.is_prime and p.divides(x * x + y * y) and (y * b).congr_mod(Nat.1, p)
        implies ((x * b) * (x * b)).congr_mod(p - Nat.1, p)
} by {
    if p.is_prime and p.divides(x * x + y * y) and (y * b).congr_mod(Nat.1, p) {
        p.is_prime
        p.divides(x * x + y * y)
        (y * b).congr_mod(Nat.1, p)
        // p | x² + y² gives (x² + y²) ≡ 0 (mod p).
        congr_mod_zero_of_divides(p, x * x + y * y)
        (x * x + y * y).congr_mod(Nat.0, p)
        // (y·b)² ≡ 1 (mod p), i.e. y²·b² ≡ 1 (mod p).
        congr_mod_pow(y * b, Nat.1, p, Nat.2)
        (y * b).pow(Nat.2).congr_mod(Nat.1.pow(Nat.2), p)
        sq_eq_mul(y * b)
        (y * b).pow(Nat.2) = (y * b) * (y * b)
        (y * b) * (y * b) = y * y * (b * b)
        (y * y * (b * b)).congr_mod(Nat.1, p)
        // (x² + y²)·b² ≡ 0 (mod p), i.e. x²·b² + y²·b² ≡ 0 (mod p).
        congr_mod_refl(b * b, p)
        (b * b).congr_mod(b * b, p)
        congr_mod_mul(x * x + y * y, Nat.0, b * b, b * b, p)
        ((x * x + y * y) * (b * b)).congr_mod(Nat.0 * (b * b), p)
        distrib_left(x * x, y * y, b * b)
        (x * x + y * y) * (b * b) = x * x * (b * b) + y * y * (b * b)
        (x * x * (b * b) + y * y * (b * b)).congr_mod(Nat.0, p)
        // x²·b² + 1 ≡ 0 (mod p) by replacing y²·b² with one.
        congr_mod_refl(x * x * (b * b), p)
        (x * x * (b * b)).congr_mod(x * x * (b * b), p)
        congr_mod_add(x * x * (b * b), y * y * (b * b), x * x * (b * b), Nat.1, p)
        (x * x * (b * b) + y * y * (b * b)).congr_mod(x * x * (b * b) + Nat.1, p)
        congr_mod_symm(x * x * (b * b) + y * y * (b * b), x * x * (b * b) + Nat.1, p)
        (x * x * (b * b) + Nat.1).congr_mod(x * x * (b * b) + y * y * (b * b), p)
        congr_mod_trans(x * x * (b * b) + Nat.1, x * x * (b * b) + y * y * (b * b), Nat.0, p)
        (x * x * (b * b) + Nat.1).congr_mod(Nat.0, p)
        // (p - 1) + 1 = p ≡ 0 (mod p), so x²·b² + 1 ≡ (p - 1) + 1 (mod p).
        Nat.1 < p
        lt_imp_lte(Nat.1, p)
        Nat.1 <= p
        add_sub(Nat.1, p)
        (p - Nat.1) + Nat.1 = p
        divides_self(p)
        congr_mod_zero_of_divides(p, p)
        p.congr_mod(Nat.0, p)
        ((p - Nat.1) + Nat.1).congr_mod(Nat.0, p)
        congr_mod_symm((p - Nat.1) + Nat.1, Nat.0, p)
        Nat.0.congr_mod((p - Nat.1) + Nat.1, p)
        congr_mod_trans(x * x * (b * b) + Nat.1, Nat.0, (p - Nat.1) + Nat.1, p)
        (x * x * (b * b) + Nat.1).congr_mod((p - Nat.1) + Nat.1, p)
        // Cancel the added one: x²·b² ≡ p - 1 (mod p).
        p != Nat.0
        congr_mod_add_cancel_right_pos(x * x * (b * b), p - Nat.1, Nat.1, p)
        (x * x * (b * b)).congr_mod(p - Nat.1, p)
        // x²·b² = (x·b)².
        (x * b) * (x * b) = x * x * (b * b)
        ((x * b) * (x * b)).congr_mod(p - Nat.1, p)
    }
}

/// The two-squares descent divides the second coordinate: a prime p ≡ 3
/// (mod 4) dividing x² + y² divides y.
///
/// If p ∤ y, y is coprime to p and has an inverse b modulo p; the inverse
/// step shows (x·b)² ≡ p - 1 (mod p), so p - 1 is a quadratic residue modulo
/// p.  But by the first supplement to quadratic reciprocity, p - 1 is a
/// quadratic residue modulo p exactly when p ≡ 1 (mod 4), contradicting
/// p ≡ 3 (mod 4).
theorem two_squares_descent_imp_divides_y(p: Nat, x: Nat, y: Nat) {
    p.is_prime and p.congr_mod(Nat.3, Nat.4) and p.divides(x * x + y * y)
        implies p.divides(y)
} by {
    if p.is_prime and p.congr_mod(Nat.3, Nat.4) and p.divides(x * x + y * y) {
        p.is_prime
        p.congr_mod(Nat.3, Nat.4)
        p.divides(x * x + y * y)
        congr_three_mod_four_imp_half_decomp(p)
        let q: Nat satisfy { p = Nat.2 * (Nat.2 * q + Nat.1) + Nat.1 }
        congr_three_mod_four_imp_not_congr_one_mod_four(p)
        not p.congr_mod(Nat.1, Nat.4)
        if p.divides(y) {
            p.divides(y)
        } else {
            // y is coprime to p and has an inverse b modulo p.
            if y.coprime(p) {
                y.coprime(p)
            } else {
                not_coprime_imp_divides_prime(p, y)
                p.divides(y)
                false
            }
            y.coprime(p)
            Nat.1 < p
            p != Nat.0
            nat_modular_inverse_exists_pos(y, p)
            let b: Nat satisfy { (y * b).congr_mod(Nat.1, p) }
            // (x·b)² ≡ p - 1 (mod p), so p - 1 is a quadratic residue.
            two_squares_descent_inverse_step(p, x, y, b)
            ((x * b) * (x * b)).congr_mod(p - Nat.1, p)
            sq_eq_mul(x * b)
            (x * b).pow(Nat.2) = (x * b) * (x * b)
            (x * b).pow(Nat.2).congr_mod(p - Nat.1, p)
            is_quadratic_residue_mod(p - Nat.1, p) =
                exists(w: Nat) { w.pow(Nat.2).congr_mod(p - Nat.1, p) }
            exists(w: Nat) { w.pow(Nat.2).congr_mod(p - Nat.1, p) }
            is_quadratic_residue_mod(p - Nat.1, p)
            // ... but p ≢ 1 (mod 4) makes p - 1 a quadratic non-residue.
            prime_pred_quadratic_residue_iff_congr_one_mod_four(p, Nat.2 * q + Nat.1)
            is_quadratic_residue_mod(p - Nat.1, p) = p.congr_mod(Nat.1, Nat.4)
            p.congr_mod(Nat.1, Nat.4)
            not p.congr_mod(Nat.1, Nat.4)
            false
        }
        p.divides(y)
    }
}

/// The two-squares descent: if a prime p ≡ 3 (mod 4) divides x² + y², then p
/// divides both x and y.  This is the descent step behind the two-squares
/// theorem (sum_of_two_squares.ac): it forces every prime divisor of a sum of
/// two squares to be 2 or 1 (mod 4).
theorem two_squares_descent(p: Nat, x: Nat, y: Nat) {
    p.is_prime and p.congr_mod(Nat.3, Nat.4) and p.divides(x * x + y * y)
        implies p.divides(x) and p.divides(y)
} by {
    if p.is_prime and p.congr_mod(Nat.3, Nat.4) and p.divides(x * x + y * y) {
        p.is_prime
        p.divides(x * x + y * y)
        // p | y by the descent above.
        two_squares_descent_imp_divides_y(p, x, y)
        p.divides(y)
        // p | x from p | x² + y² and p | y².
        divides_mul(y, y, p)
        p.divides(y * y)
        divides_sub(x * x + y * y, y * y, p)
        p.divides((x * x + y * y) - y * y)
        add_comm(x * x, y * y)
        x * x + y * y = y * y + x * x
        add_imp_sub(y * y, x * x, x * x + y * y)
        (x * x + y * y) - y * y = x * x
        p.divides(x * x)
        // p | x·x, hence p | x (p is prime).
        gcd_of_prime(p, x)
        if p.gcd(x) = Nat.1 {
            p.coprime(x)
            coprime_divides_of_divides_mul(p, x, x)
            p.divides(x)
        } else {
            p.divides(x)
        }
        p.divides(x)
        p.divides(x) and p.divides(y)
    }
}
