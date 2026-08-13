from nat import Nat, add_assoc, add_cancels_left, add_comm, add_one_right,
    add_suc_right, distrib_left, distrib_right, from_nat_one, from_nat_zero,
    lt_and_lte,
    lt_imp_lte_suc, lt_mul_both, lt_suc, lte_add_left, lte_add_right,
    lte_mul_both, lte_trans, mul_assoc, mul_comm, mul_suc_right,
    mul_zero_left, mul_zero_right, add_imp_sub, add_imp_sub_left,
    divides_mul, divides_sub, gcd_divides_left, gcd_divides_right,
    mul_to_one
from int import Int, add_from_nat, mul_from_nat, neg_sub
from rat import Rat, cross_mul_lt, from_nat_add, from_nat_mul,
    nat_lt_imp_rat_lt
from pair import Pair, pair_new_first, pair_new_second
from algebra.ring.ring import alternating_sign, alternating_sign_zero, alternating_sign_suc
from number_theory.coprime import nat_divides_one_imp_one

numerals Nat

/// The pair of consecutive values obtained after a finite number of
/// continued-fraction recurrence steps.
define continued_fraction_recurrence_state(
    coefficients: Nat -> Nat, n: Nat, previous: Nat, current: Nat
) -> Pair[Nat, Nat] {
    match n {
        Nat.zero {
            Pair.new(previous, current)
        }
        Nat.suc(k) {
            let state = continued_fraction_recurrence_state(
                coefficients, k, previous, current)
            Pair.new(state.second,
                state.second * coefficients(k) + state.first)
        }
    }
}

/// The numerator of the convergent at an index.
define continued_fraction_convergent_numerator(
    coefficients: Nat -> Nat, n: Nat
) -> Nat {
    continued_fraction_recurrence_state(
        coefficients, n.suc, Nat.0, Nat.1).second
}

/// The denominator of the convergent at an index.
define continued_fraction_convergent_denominator(
    coefficients: Nat -> Nat, n: Nat
) -> Nat {
    continued_fraction_recurrence_state(
        coefficients, n.suc, Nat.1, Nat.0).second
}

/// The numerator-denominator pair of the convergent at an index.
define continued_fraction_convergent_sequence(
    coefficients: Nat -> Nat, n: Nat
) -> Pair[Nat, Nat] {
    Pair.new(
        continued_fraction_convergent_numerator(coefficients, n),
        continued_fraction_convergent_denominator(coefficients, n))
}

/// True if every continued-fraction coefficient after the integral part is
/// positive.
define positive_continued_fraction_sequence_tail(
    coefficients: Nat -> Nat
) -> Bool {
    forall(n: Nat) {
        Nat.0 < coefficients(n.suc)
    }
}

/// The rational value of the convergent at an index.
define continued_fraction_convergent_value(
    coefficients: Nat -> Nat, n: Nat
) -> Rat {
    Rat.from_nat(continued_fraction_convergent_numerator(coefficients, n)) /
        Rat.from_nat(continued_fraction_convergent_denominator(coefficients, n))
}

/// The signed determinant of the numerator and denominator recurrence states.
define continued_fraction_recurrence_determinant(
    coefficients: Nat -> Nat, n: Nat
) -> Int {
    let numerator_state = continued_fraction_recurrence_state(
        coefficients, n, Nat.0, Nat.1)
    let denominator_state = continued_fraction_recurrence_state(
        coefficients, n, Nat.1, Nat.0)
    Int.from_nat(numerator_state.second) *
        Int.from_nat(denominator_state.first) -
        Int.from_nat(numerator_state.first) *
            Int.from_nat(denominator_state.second)
}

/// The signed determinant of two adjacent convergents.
define continued_fraction_adjacent_convergent_determinant(
    coefficients: Nat -> Nat, n: Nat
) -> Int {
    Int.from_nat(continued_fraction_convergent_numerator(
        coefficients, n.suc)) *
        Int.from_nat(continued_fraction_convergent_denominator(
            coefficients, n)) -
    Int.from_nat(continued_fraction_convergent_numerator(
        coefficients, n)) *
        Int.from_nat(continued_fraction_convergent_denominator(
            coefficients, n.suc))
}

/// The zero-step recurrence state is its pair of initial values.
theorem continued_fraction_recurrence_state_zero(
    coefficients: Nat -> Nat, previous: Nat, current: Nat
) {
    continued_fraction_recurrence_state(
        coefficients, Nat.0, previous, current) =
        Pair.new(previous, current)
}

/// A successor recurrence state shifts the current value and performs one
/// continued-fraction recurrence step.
theorem continued_fraction_recurrence_state_suc(
    coefficients: Nat -> Nat, n: Nat, previous: Nat, current: Nat
) {
    continued_fraction_recurrence_state(
        coefficients, n.suc, previous, current) =
        Pair.new(
            continued_fraction_recurrence_state(
                coefficients, n, previous, current).second,
            continued_fraction_recurrence_state(
                coefficients, n, previous, current).second * coefficients(n) +
            continued_fraction_recurrence_state(
                coefficients, n, previous, current).first)
}

/// The first component of a successor recurrence state is the previous
/// current component.
theorem continued_fraction_recurrence_state_suc_first(
    coefficients: Nat -> Nat, n: Nat, previous: Nat, current: Nat
) {
    continued_fraction_recurrence_state(
        coefficients, n.suc, previous, current).first =
        continued_fraction_recurrence_state(
            coefficients, n, previous, current).second
} by {
    continued_fraction_recurrence_state_suc(
        coefficients, n, previous, current)
    pair_new_first(
        continued_fraction_recurrence_state(
            coefficients, n, previous, current).second,
        continued_fraction_recurrence_state(
            coefficients, n, previous, current).second * coefficients(n) +
            continued_fraction_recurrence_state(
                coefficients, n, previous, current).first)
}

/// The second component of a successor recurrence state is the next
/// recurrence value.
theorem continued_fraction_recurrence_state_suc_second(
    coefficients: Nat -> Nat, n: Nat, previous: Nat, current: Nat
) {
    continued_fraction_recurrence_state(
        coefficients, n.suc, previous, current).second =
        continued_fraction_recurrence_state(
            coefficients, n, previous, current).second * coefficients(n) +
        continued_fraction_recurrence_state(
            coefficients, n, previous, current).first
} by {
    continued_fraction_recurrence_state_suc(
        coefficients, n, previous, current)
    pair_new_second(
        continued_fraction_recurrence_state(
            coefficients, n, previous, current).second,
        continued_fraction_recurrence_state(
            coefficients, n, previous, current).second * coefficients(n) +
            continued_fraction_recurrence_state(
                coefficients, n, previous, current).first)
}

/// The first convergent has its first coefficient as numerator.
theorem continued_fraction_convergent_numerator_zero(
    coefficients: Nat -> Nat
) {
    continued_fraction_convergent_numerator(coefficients, Nat.0) =
        coefficients(Nat.0)
} by {
    continued_fraction_recurrence_state_suc_second(
        coefficients, Nat.0, Nat.0, Nat.1)
    continued_fraction_recurrence_state_zero(
        coefficients, Nat.0, Nat.1)
    pair_new_first(Nat.0, Nat.1)
    pair_new_second(Nat.0, Nat.1)
}

/// The first convergent has denominator one.
theorem continued_fraction_convergent_denominator_zero(
    coefficients: Nat -> Nat
) {
    continued_fraction_convergent_denominator(coefficients, Nat.0) = Nat.1
} by {
    continued_fraction_recurrence_state_suc_second(
        coefficients, Nat.0, Nat.1, Nat.0)
    continued_fraction_recurrence_state_zero(
        coefficients, Nat.1, Nat.0)
    pair_new_first(Nat.1, Nat.0)
    pair_new_second(Nat.1, Nat.0)
    mul_zero_left(coefficients(Nat.0))
    Nat.0 * coefficients(Nat.0) + Nat.1 = Nat.1
}

/// A successor convergent numerator is obtained by the continued-fraction
/// recurrence.
theorem continued_fraction_convergent_numerator_suc(
    coefficients: Nat -> Nat, n: Nat
) {
    continued_fraction_convergent_numerator(coefficients, n.suc) =
        continued_fraction_convergent_numerator(coefficients, n) *
            coefficients(n.suc) +
        continued_fraction_recurrence_state(
            coefficients, n.suc, Nat.0, Nat.1).first
} by {
    continued_fraction_recurrence_state_suc_second(
        coefficients, n.suc, Nat.0, Nat.1)
}

/// A successor convergent denominator is obtained by the continued-fraction
/// recurrence.
theorem continued_fraction_convergent_denominator_suc(
    coefficients: Nat -> Nat, n: Nat
) {
    continued_fraction_convergent_denominator(coefficients, n.suc) =
        continued_fraction_convergent_denominator(coefficients, n) *
            coefficients(n.suc) +
        continued_fraction_recurrence_state(
            coefficients, n.suc, Nat.1, Nat.0).first
} by {
    continued_fraction_recurrence_state_suc_second(
        coefficients, n.suc, Nat.1, Nat.0)
}

/// Every convergent denominator is positive when all coefficients after the
/// integral part are positive.
theorem continued_fraction_convergent_denominator_positive(
    coefficients: Nat -> Nat, n: Nat
) {
    positive_continued_fraction_sequence_tail(coefficients) implies
        Nat.0 < continued_fraction_convergent_denominator(coefficients, n)
} by {
    define p(k: Nat) -> Bool {
        Nat.0 < continued_fraction_convergent_denominator(coefficients, k)
    }
    if positive_continued_fraction_sequence_tail(coefficients) {
        continued_fraction_convergent_denominator_zero(coefficients)
        Nat.0 < Nat.1
        p(Nat.0)
        forall(k: Nat) {
            if p(k) {
                positive_continued_fraction_sequence_tail(coefficients)
                Nat.0 < coefficients(k.suc)
                continued_fraction_convergent_denominator_suc(coefficients, k)
                let d = continued_fraction_convergent_denominator(
                    coefficients, k)
                let a = coefficients(k.suc)
                let previous = continued_fraction_recurrence_state(
                    coefficients, k.suc, Nat.1, Nat.0).first
                Nat.0 < d
                d != Nat.0
                lt_mul_both(d, Nat.0, a)
                d * Nat.0 < d * a
                mul_zero_right(d)
                Nat.0 < d * a
                Nat.0 <= previous
                lte_add_left(d * a, Nat.0, previous)
                d * a + Nat.0 <= d * a + previous
                d * a <= d * a + previous
                lt_and_lte(Nat.0, d * a, d * a + previous)
                Nat.0 < d * a + previous
                continued_fraction_convergent_denominator(
                    coefficients, k.suc) = d * a + previous
                p(k.suc)
            }
        }
        p(Nat.0) and forall(k: Nat) { p(k) implies p(k.suc) }
        Nat.induction(p)
        forall(k: Nat) { p(k) }
        p(n)
    }
}

/// A convergent pair has the sequence numerator as its first component.
theorem continued_fraction_convergent_sequence_first(
    coefficients: Nat -> Nat, n: Nat
) {
    continued_fraction_convergent_sequence(coefficients, n).first =
        continued_fraction_convergent_numerator(coefficients, n)
} by {
    pair_new_first(
        continued_fraction_convergent_numerator(coefficients, n),
        continued_fraction_convergent_denominator(coefficients, n))
}

/// A convergent pair has the sequence denominator as its second component.
theorem continued_fraction_convergent_sequence_second(
    coefficients: Nat -> Nat, n: Nat
) {
    continued_fraction_convergent_sequence(coefficients, n).second =
        continued_fraction_convergent_denominator(coefficients, n)
} by {
    pair_new_second(
        continued_fraction_convergent_numerator(coefficients, n),
        continued_fraction_convergent_denominator(coefficients, n))
}

/// The initial recurrence determinant is one.
theorem continued_fraction_recurrence_determinant_zero(
    coefficients: Nat -> Nat
) {
    continued_fraction_recurrence_determinant(coefficients, Nat.0) = Int.1
} by {
    continued_fraction_recurrence_state_zero(
        coefficients, Nat.0, Nat.1)
    continued_fraction_recurrence_state_zero(
        coefficients, Nat.1, Nat.0)
    pair_new_first(Nat.0, Nat.1)
    pair_new_second(Nat.0, Nat.1)
    pair_new_first(Nat.1, Nat.0)
    pair_new_second(Nat.1, Nat.0)
    continued_fraction_recurrence_determinant(coefficients, Nat.0) =
        Int.from_nat(Nat.1) * Int.from_nat(Nat.1) -
            Int.from_nat(Nat.0) * Int.from_nat(Nat.0)
    from_nat_one[Int]
    from_nat_zero[Int]
    Int.from_nat(Nat.1) = Int.1
    Int.from_nat(Nat.0) = Int.0
    Int.1 * Int.1 - Int.0 * Int.0 = Int.1
    Int.from_nat(Nat.1) * Int.from_nat(Nat.1) -
        Int.from_nat(Nat.0) * Int.from_nat(Nat.0) = Int.1
}

/// Each recurrence step reverses the sign of the recurrence determinant.
theorem continued_fraction_recurrence_determinant_suc(
    coefficients: Nat -> Nat, n: Nat
) {
    continued_fraction_recurrence_determinant(coefficients, n.suc) =
        -continued_fraction_recurrence_determinant(coefficients, n)
} by {
    let numerator_state = continued_fraction_recurrence_state(
        coefficients, n, Nat.0, Nat.1)
    let denominator_state = continued_fraction_recurrence_state(
        coefficients, n, Nat.1, Nat.0)
    let a = coefficients(n)
    let pn = numerator_state.first
    let cn = numerator_state.second
    let pd = denominator_state.first
    let cd = denominator_state.second
    let ai = Int.from_nat(a)
    let pni = Int.from_nat(pn)
    let cni = Int.from_nat(cn)
    let pdi = Int.from_nat(pd)
    let cdi = Int.from_nat(cd)
    continued_fraction_recurrence_state_suc_first(
        coefficients, n, Nat.0, Nat.1)
    continued_fraction_recurrence_state_suc_second(
        coefficients, n, Nat.0, Nat.1)
    continued_fraction_recurrence_state_suc_first(
        coefficients, n, Nat.1, Nat.0)
    continued_fraction_recurrence_state_suc_second(
        coefficients, n, Nat.1, Nat.0)
    continued_fraction_recurrence_determinant(coefficients, n.suc) =
        Int.from_nat(cn * a + pn) * Int.from_nat(cd) -
            Int.from_nat(cn) * Int.from_nat(cd * a + pd)
    add_from_nat(cn * a, pn)
    add_from_nat(cd * a, pd)
    mul_from_nat(cn, a)
    mul_from_nat(cd, a)
    mul_from_nat(cn * a + pn, cd)
    mul_from_nat(cn, cd * a + pd)
    continued_fraction_recurrence_determinant(coefficients, n) =
        Int.from_nat(cn) * Int.from_nat(pd) -
            Int.from_nat(pn) * Int.from_nat(cd)
    Int.from_nat(cn * a + pn) =
        Int.from_nat(cn) * Int.from_nat(a) + Int.from_nat(pn)
    Int.from_nat(cd * a + pd) =
        Int.from_nat(cd) * Int.from_nat(a) + Int.from_nat(pd)
    Int.from_nat(cn * a + pn) * Int.from_nat(cd) -
        Int.from_nat(cn) * Int.from_nat(cd * a + pd) =
        (Int.from_nat(cn) * Int.from_nat(a) + Int.from_nat(pn)) *
            Int.from_nat(cd) -
        Int.from_nat(cn) *
            (Int.from_nat(cd) * Int.from_nat(a) + Int.from_nat(pd))
    (cni * ai + pni) * cdi = cni * ai * cdi + pni * cdi
    cni * (cdi * ai + pdi) = cni * (cdi * ai) + cni * pdi
    cni * ai * cdi = cni * (cdi * ai)
    (cni * ai + pni) * cdi - cni * (cdi * ai + pdi) =
        (cni * ai * cdi + pni * cdi) -
            (cni * ai * cdi + cni * pdi)
    (cni * ai * cdi + pni * cdi) -
        (cni * ai * cdi + cni * pdi) =
        pni * cdi - cni * pdi
    (Int.from_nat(cn) * Int.from_nat(a) + Int.from_nat(pn)) *
        Int.from_nat(cd) -
        Int.from_nat(cn) *
            (Int.from_nat(cd) * Int.from_nat(a) + Int.from_nat(pd)) =
        Int.from_nat(pn) * Int.from_nat(cd) -
            Int.from_nat(cn) * Int.from_nat(pd)
    neg_sub(
        Int.from_nat(cn) * Int.from_nat(pd),
        Int.from_nat(pn) * Int.from_nat(cd))
    Int.from_nat(pn) * Int.from_nat(cd) -
        Int.from_nat(cn) * Int.from_nat(pd) =
        -(Int.from_nat(cn) * Int.from_nat(pd) -
            Int.from_nat(pn) * Int.from_nat(cd))
    Int.from_nat(cn * a + pn) * Int.from_nat(cd) -
        Int.from_nat(cn) * Int.from_nat(cd * a + pd) =
        -(Int.from_nat(cn) * Int.from_nat(pd) -
            Int.from_nat(pn) * Int.from_nat(cd))
}

/// The recurrence determinant is the alternating sign at the number of
/// recurrence steps.
theorem continued_fraction_recurrence_determinant_eq_alternating_sign(
    coefficients: Nat -> Nat, n: Nat
) {
    continued_fraction_recurrence_determinant(coefficients, n) =
        alternating_sign[Int](n)
} by {
    define p(k: Nat) -> Bool {
        continued_fraction_recurrence_determinant(coefficients, k) =
            alternating_sign[Int](k)
    }
    continued_fraction_recurrence_determinant_zero(coefficients)
    alternating_sign_zero[Int]
    p(Nat.0)
    forall(k: Nat) {
        if p(k) {
            continued_fraction_recurrence_determinant_suc(coefficients, k)
            alternating_sign_suc[Int](k)
            continued_fraction_recurrence_determinant(coefficients, k.suc) =
                -continued_fraction_recurrence_determinant(coefficients, k)
            continued_fraction_recurrence_determinant(coefficients, k) =
                alternating_sign[Int](k)
            continued_fraction_recurrence_determinant(coefficients, k.suc) =
                -alternating_sign[Int](k)
            continued_fraction_recurrence_determinant(coefficients, k.suc) =
                alternating_sign[Int](k.suc)
            p(k.suc)
        }
    }
    p(Nat.0) and forall(k: Nat) { p(k) implies p(k.suc) }
    Nat.induction(p)
    forall(k: Nat) { p(k) }
    p(n)
}

/// Adjacent convergents have determinant equal to the alternating sign of the
/// lower index.
theorem continued_fraction_adjacent_convergent_determinant_identity(
    coefficients: Nat -> Nat, n: Nat
) {
    continued_fraction_adjacent_convergent_determinant(coefficients, n) =
        alternating_sign[Int](n)
} by {
    let numerator_state = continued_fraction_recurrence_state(
        coefficients, n.suc, Nat.0, Nat.1)
    let denominator_state = continued_fraction_recurrence_state(
        coefficients, n.suc, Nat.1, Nat.0)
    let a = coefficients(n.suc)
    let pn = numerator_state.first
    let cn = numerator_state.second
    let pd = denominator_state.first
    let cd = denominator_state.second
    continued_fraction_convergent_numerator_suc(coefficients, n)
    continued_fraction_convergent_denominator_suc(coefficients, n)
    continued_fraction_convergent_numerator(coefficients, n) = cn
    continued_fraction_convergent_denominator(coefficients, n) = cd
    continued_fraction_convergent_numerator(coefficients, n.suc) =
        cn * a + pn
    continued_fraction_convergent_denominator(coefficients, n.suc) =
        cd * a + pd
    continued_fraction_adjacent_convergent_determinant(coefficients, n) =
        Int.from_nat(cn * a + pn) * Int.from_nat(cd) -
            Int.from_nat(cn) * Int.from_nat(cd * a + pd)
    add_from_nat(cn * a, pn)
    add_from_nat(cd * a, pd)
    mul_from_nat(cn, a)
    mul_from_nat(cd, a)
    Int.from_nat(cn * a + pn) =
        Int.from_nat(cn) * Int.from_nat(a) + Int.from_nat(pn)
    Int.from_nat(cd * a + pd) =
        Int.from_nat(cd) * Int.from_nat(a) + Int.from_nat(pd)
    let ai = Int.from_nat(a)
    let pni = Int.from_nat(pn)
    let cni = Int.from_nat(cn)
    let pdi = Int.from_nat(pd)
    let cdi = Int.from_nat(cd)
    (cni * ai + pni) * cdi = cni * ai * cdi + pni * cdi
    cni * (cdi * ai + pdi) = cni * (cdi * ai) + cni * pdi
    cni * ai * cdi = cni * (cdi * ai)
    (cni * ai + pni) * cdi - cni * (cdi * ai + pdi) =
        (cni * ai * cdi + pni * cdi) -
            (cni * ai * cdi + cni * pdi)
    (cni * ai * cdi + pni * cdi) -
        (cni * ai * cdi + cni * pdi) =
        pni * cdi - cni * pdi
    neg_sub(cni * pdi, pni * cdi)
    pni * cdi - cni * pdi = -(cni * pdi - pni * cdi)
    continued_fraction_recurrence_determinant(coefficients, n.suc) =
        cni * pdi - pni * cdi
    continued_fraction_adjacent_convergent_determinant(coefficients, n) =
        -continued_fraction_recurrence_determinant(coefficients, n.suc)
    continued_fraction_recurrence_determinant_suc(coefficients, n)
    -continued_fraction_recurrence_determinant(coefficients, n.suc) =
        continued_fraction_recurrence_determinant(coefficients, n)
    continued_fraction_recurrence_determinant_eq_alternating_sign(
        coefficients, n)
}

/// A positive adjacent determinant places the lower cross product exactly one
/// below the upper cross product.
theorem continued_fraction_adjacent_cross_product_of_positive_sign(
    coefficients: Nat -> Nat, n: Nat
) {
    alternating_sign[Int](n) = Int.1 implies
        continued_fraction_convergent_numerator(coefficients, n) *
                continued_fraction_convergent_denominator(
                    coefficients, n.suc) + Nat.1 =
            continued_fraction_convergent_numerator(coefficients, n.suc) *
                continued_fraction_convergent_denominator(coefficients, n)
} by {
    if alternating_sign[Int](n) = Int.1 {
        let lower = continued_fraction_convergent_numerator(coefficients, n) *
            continued_fraction_convergent_denominator(coefficients, n.suc)
        let upper = continued_fraction_convergent_numerator(
            coefficients, n.suc) *
            continued_fraction_convergent_denominator(coefficients, n)
        continued_fraction_adjacent_convergent_determinant_identity(
            coefficients, n)
        mul_from_nat(
            continued_fraction_convergent_numerator(coefficients, n.suc),
            continued_fraction_convergent_denominator(coefficients, n))
        mul_from_nat(
            continued_fraction_convergent_numerator(coefficients, n),
            continued_fraction_convergent_denominator(coefficients, n.suc))
        Int.from_nat(upper) - Int.from_nat(lower) = Int.1
        add_from_nat(lower, Nat.1)
        from_nat_one[Int]
        Int.from_nat(lower + Nat.1) =
            Int.from_nat(lower) + Int.1
        Int.from_nat(lower) + Int.1 = Int.from_nat(upper)
        Int.from_nat(lower + Nat.1) = Int.from_nat(upper)
        lower + Nat.1 = upper
        continued_fraction_convergent_numerator(coefficients, n) *
                continued_fraction_convergent_denominator(
                    coefficients, n.suc) + Nat.1 =
            continued_fraction_convergent_numerator(coefficients, n.suc) *
                continued_fraction_convergent_denominator(coefficients, n)
    }
}

/// A negative adjacent determinant places the upper cross product exactly one
/// below the lower cross product.
theorem continued_fraction_adjacent_cross_product_of_negative_sign(
    coefficients: Nat -> Nat, n: Nat
) {
    alternating_sign[Int](n) = -Int.1 implies
        continued_fraction_convergent_numerator(coefficients, n.suc) *
                continued_fraction_convergent_denominator(coefficients, n) +
                Nat.1 =
            continued_fraction_convergent_numerator(coefficients, n) *
                continued_fraction_convergent_denominator(
                    coefficients, n.suc)
} by {
    if alternating_sign[Int](n) = -Int.1 {
        let lower = continued_fraction_convergent_numerator(
            coefficients, n.suc) *
            continued_fraction_convergent_denominator(coefficients, n)
        let upper = continued_fraction_convergent_numerator(coefficients, n) *
            continued_fraction_convergent_denominator(coefficients, n.suc)
        continued_fraction_adjacent_convergent_determinant_identity(
            coefficients, n)
        mul_from_nat(
            continued_fraction_convergent_numerator(coefficients, n.suc),
            continued_fraction_convergent_denominator(coefficients, n))
        mul_from_nat(
            continued_fraction_convergent_numerator(coefficients, n),
            continued_fraction_convergent_denominator(coefficients, n.suc))
        Int.from_nat(lower) - Int.from_nat(upper) = -Int.1
        add_from_nat(lower, Nat.1)
        from_nat_one[Int]
        Int.from_nat(lower + Nat.1) =
            Int.from_nat(lower) + Int.1
        Int.from_nat(lower) + Int.1 = Int.from_nat(upper)
        Int.from_nat(lower + Nat.1) = Int.from_nat(upper)
        lower + Nat.1 = upper
        continued_fraction_convergent_numerator(coefficients, n.suc) *
                continued_fraction_convergent_denominator(coefficients, n) +
                Nat.1 =
            continued_fraction_convergent_numerator(coefficients, n) *
                continued_fraction_convergent_denominator(
                    coefficients, n.suc)
    }
}

/// A natural number one step above another is strictly larger.
theorem continued_fraction_add_one_eq_imp_lt(left: Nat, right: Nat) {
    left + Nat.1 = right implies left < right
} by {
    if left + Nat.1 = right {
        lt_suc(left)
        add_one_right(left)
        left < left.suc
        left.suc = right
        left < right
    }
}

/// A positive adjacent determinant makes the lower cross product strictly
/// smaller than the upper cross product.
theorem continued_fraction_adjacent_cross_product_lt_of_positive_sign(
    coefficients: Nat -> Nat, n: Nat
) {
    alternating_sign[Int](n) = Int.1 implies
        continued_fraction_convergent_numerator(coefficients, n) *
            continued_fraction_convergent_denominator(coefficients, n.suc) <
        continued_fraction_convergent_numerator(coefficients, n.suc) *
            continued_fraction_convergent_denominator(coefficients, n)
} by {
    if alternating_sign[Int](n) = Int.1 {
        continued_fraction_adjacent_cross_product_of_positive_sign(
            coefficients, n)
        continued_fraction_add_one_eq_imp_lt(
            continued_fraction_convergent_numerator(coefficients, n) *
                continued_fraction_convergent_denominator(
                    coefficients, n.suc),
            continued_fraction_convergent_numerator(coefficients, n.suc) *
                continued_fraction_convergent_denominator(coefficients, n))
    }
}

/// A negative adjacent determinant makes the next-current cross product
/// strictly smaller than the current-next cross product.
theorem continued_fraction_adjacent_cross_product_lt_of_negative_sign(
    coefficients: Nat -> Nat, n: Nat
) {
    alternating_sign[Int](n) = -Int.1 implies
        continued_fraction_convergent_numerator(coefficients, n.suc) *
            continued_fraction_convergent_denominator(coefficients, n) <
        continued_fraction_convergent_numerator(coefficients, n) *
            continued_fraction_convergent_denominator(coefficients, n.suc)
} by {
    if alternating_sign[Int](n) = -Int.1 {
        continued_fraction_adjacent_cross_product_of_negative_sign(
            coefficients, n)
        continued_fraction_add_one_eq_imp_lt(
            continued_fraction_convergent_numerator(coefficients, n.suc) *
                continued_fraction_convergent_denominator(coefficients, n),
            continued_fraction_convergent_numerator(coefficients, n) *
                continued_fraction_convergent_denominator(
                    coefficients, n.suc))
    }
}

/// Cross multiplication compares rational quotients of natural numbers with
/// positive denominators.
theorem continued_fraction_nat_ratio_lt_of_cross_product(
    left_numerator: Nat, left_denominator: Nat,
    right_numerator: Nat, right_denominator: Nat
) {
    Nat.0 < left_denominator and Nat.0 < right_denominator and
        left_numerator * right_denominator <
            right_numerator * left_denominator implies
        Rat.from_nat(left_numerator) / Rat.from_nat(left_denominator) <
            Rat.from_nat(right_numerator) / Rat.from_nat(right_denominator)
} by {
    if Nat.0 < left_denominator and Nat.0 < right_denominator and
            left_numerator * right_denominator <
                right_numerator * left_denominator {
        nat_lt_imp_rat_lt(Nat.0, left_denominator)
        nat_lt_imp_rat_lt(Nat.0, right_denominator)
        from_nat_zero[Rat]
        Rat.0 < Rat.from_nat(left_denominator)
        Rat.0 < Rat.from_nat(right_denominator)
        Rat.from_nat(left_denominator).is_positive
        Rat.from_nat(right_denominator).is_positive
        nat_lt_imp_rat_lt(
            left_numerator * right_denominator,
            right_numerator * left_denominator)
        from_nat_mul(left_numerator, right_denominator)
        from_nat_mul(right_numerator, left_denominator)
        Rat.from_nat(left_numerator) * Rat.from_nat(right_denominator) < Rat.from_nat(right_numerator) * Rat.from_nat(left_denominator)
        cross_mul_lt(
            Rat.from_nat(left_numerator), Rat.from_nat(left_denominator),
            Rat.from_nat(right_numerator), Rat.from_nat(right_denominator))
        Rat.from_nat(left_numerator) / Rat.from_nat(left_denominator) < Rat.from_nat(right_numerator) / Rat.from_nat(right_denominator)
    }
}

/// A strict cross-product comparison gives the corresponding comparison of
/// adjacent convergent values.
theorem continued_fraction_convergent_value_lt_next_of_cross_product(
    coefficients: Nat -> Nat, n: Nat
) {
    positive_continued_fraction_sequence_tail(coefficients) and
        continued_fraction_convergent_numerator(coefficients, n) *
                continued_fraction_convergent_denominator(
                    coefficients, n.suc) <
            continued_fraction_convergent_numerator(coefficients, n.suc) *
                continued_fraction_convergent_denominator(coefficients, n)
        implies continued_fraction_convergent_value(coefficients, n) <
            continued_fraction_convergent_value(coefficients, n.suc)
} by {
    if positive_continued_fraction_sequence_tail(coefficients) and
            continued_fraction_convergent_numerator(coefficients, n) *
                    continued_fraction_convergent_denominator(
                        coefficients, n.suc) <
                continued_fraction_convergent_numerator(coefficients, n.suc) *
                    continued_fraction_convergent_denominator(coefficients, n) {
        continued_fraction_convergent_denominator_positive(coefficients, n)
        continued_fraction_convergent_denominator_positive(coefficients, n.suc)
        continued_fraction_nat_ratio_lt_of_cross_product(
            continued_fraction_convergent_numerator(coefficients, n),
            continued_fraction_convergent_denominator(coefficients, n),
            continued_fraction_convergent_numerator(coefficients, n.suc),
            continued_fraction_convergent_denominator(coefficients, n.suc))
        continued_fraction_convergent_value(coefficients, n) =
            Rat.from_nat(continued_fraction_convergent_numerator(
                coefficients, n)) /
            Rat.from_nat(continued_fraction_convergent_denominator(
                coefficients, n))
        continued_fraction_convergent_value(coefficients, n.suc) =
            Rat.from_nat(continued_fraction_convergent_numerator(
                coefficients, n.suc)) /
            Rat.from_nat(continued_fraction_convergent_denominator(
                coefficients, n.suc))
        continued_fraction_convergent_value(coefficients, n) < continued_fraction_convergent_value(coefficients, n.suc)
    }
}

/// A reverse strict cross-product comparison gives the reverse comparison of
/// adjacent convergent values.
theorem continued_fraction_next_value_lt_of_cross_product(
    coefficients: Nat -> Nat, n: Nat
) {
    positive_continued_fraction_sequence_tail(coefficients) and
        continued_fraction_convergent_numerator(coefficients, n.suc) *
                continued_fraction_convergent_denominator(coefficients, n) <
            continued_fraction_convergent_numerator(coefficients, n) *
                continued_fraction_convergent_denominator(
                    coefficients, n.suc)
        implies continued_fraction_convergent_value(coefficients, n.suc) <
            continued_fraction_convergent_value(coefficients, n)
} by {
    if positive_continued_fraction_sequence_tail(coefficients) and
            continued_fraction_convergent_numerator(coefficients, n.suc) *
                    continued_fraction_convergent_denominator(coefficients, n) <
                continued_fraction_convergent_numerator(coefficients, n) *
                    continued_fraction_convergent_denominator(
                        coefficients, n.suc) {
        continued_fraction_convergent_denominator_positive(coefficients, n)
        continued_fraction_convergent_denominator_positive(coefficients, n.suc)
        continued_fraction_nat_ratio_lt_of_cross_product(
            continued_fraction_convergent_numerator(coefficients, n.suc),
            continued_fraction_convergent_denominator(coefficients, n.suc),
            continued_fraction_convergent_numerator(coefficients, n),
            continued_fraction_convergent_denominator(coefficients, n))
        continued_fraction_convergent_value(coefficients, n.suc) =
            Rat.from_nat(continued_fraction_convergent_numerator(
                coefficients, n.suc)) /
            Rat.from_nat(continued_fraction_convergent_denominator(
                coefficients, n.suc))
        continued_fraction_convergent_value(coefficients, n) =
            Rat.from_nat(continued_fraction_convergent_numerator(
                coefficients, n)) /
            Rat.from_nat(continued_fraction_convergent_denominator(
                coefficients, n))
        continued_fraction_convergent_value(coefficients, n.suc) < continued_fraction_convergent_value(coefficients, n)
    }
}

/// At a positive alternating sign, the next convergent is a strict upper
/// bound for the current convergent.
theorem continued_fraction_convergent_value_lt_next_of_positive_sign(
    coefficients: Nat -> Nat, n: Nat
) {
    positive_continued_fraction_sequence_tail(coefficients) and
        alternating_sign[Int](n) = Int.1 implies
        continued_fraction_convergent_value(coefficients, n) < continued_fraction_convergent_value(coefficients, n.suc)
} by {
    if positive_continued_fraction_sequence_tail(coefficients) and
            alternating_sign[Int](n) = Int.1 {
        continued_fraction_adjacent_cross_product_lt_of_positive_sign(
            coefficients, n)
        continued_fraction_convergent_value_lt_next_of_cross_product(
            coefficients, n)
    }
}

/// At a negative alternating sign, the current convergent is a strict upper
/// bound for the next convergent.
theorem continued_fraction_convergent_value_gt_next_of_negative_sign(
    coefficients: Nat -> Nat, n: Nat
) {
    positive_continued_fraction_sequence_tail(coefficients) and
        alternating_sign[Int](n) = -Int.1 implies
        continued_fraction_convergent_value(coefficients, n.suc) < continued_fraction_convergent_value(coefficients, n)
} by {
    if positive_continued_fraction_sequence_tail(coefficients) and
            alternating_sign[Int](n) = -Int.1 {
        continued_fraction_adjacent_cross_product_lt_of_negative_sign(
            coefficients, n)
        continued_fraction_next_value_lt_of_cross_product(coefficients, n)
    }
}

/// The alternating sign at twice an index is positive.
theorem continued_fraction_alternating_sign_double(n: Nat) {
    alternating_sign[Int](Nat.2 * n) = Int.1
} by {
    define p(k: Nat) -> Bool {
        alternating_sign[Int](Nat.2 * k) = Int.1
    }
    alternating_sign_zero[Int]
    Nat.2 * Nat.0 = Nat.0
    p(Nat.0)
    forall(k: Nat) {
        if p(k) {
            mul_suc_right(Nat.2, k)
            Nat.2 * k.suc = Nat.2 + Nat.2 * k
            add_comm(Nat.2, Nat.2 * k)
            Nat.2 + Nat.2 * k = Nat.2 * k + Nat.2
            add_suc_right(Nat.2 * k, Nat.1)
            Nat.2 * k + Nat.2 = (Nat.2 * k + Nat.1).suc
            add_one_right(Nat.2 * k)
            (Nat.2 * k + Nat.1).suc = (Nat.2 * k).suc.suc
            Nat.2 * k.suc = (Nat.2 * k).suc.suc
            alternating_sign_suc[Int](Nat.2 * k)
            alternating_sign_suc[Int]((Nat.2 * k).suc)
            alternating_sign[Int]((Nat.2 * k).suc) =
                -alternating_sign[Int](Nat.2 * k)
            alternating_sign[Int]((Nat.2 * k).suc.suc) =
                -alternating_sign[Int]((Nat.2 * k).suc)
            alternating_sign[Int]((Nat.2 * k).suc.suc) =
                --alternating_sign[Int](Nat.2 * k)
            --alternating_sign[Int](Nat.2 * k) =
                alternating_sign[Int](Nat.2 * k)
            alternating_sign[Int](Nat.2 * k) = Int.1
            alternating_sign[Int](Nat.2 * k.suc) = Int.1
            p(k.suc)
        }
    }
    p(Nat.0) and forall(k: Nat) { p(k) implies p(k.suc) }
    Nat.induction(p)
    forall(k: Nat) { p(k) }
    p(n)
}

/// The alternating sign immediately after twice an index is negative.
theorem continued_fraction_alternating_sign_double_suc(n: Nat) {
    alternating_sign[Int]((Nat.2 * n).suc) = -Int.1
} by {
    continued_fraction_alternating_sign_double(n)
    alternating_sign_suc[Int](Nat.2 * n)
}

/// Each even-indexed convergent is strictly below the following odd-indexed
/// convergent.
theorem continued_fraction_even_convergent_lt_following_odd(
    coefficients: Nat -> Nat, n: Nat
) {
    positive_continued_fraction_sequence_tail(coefficients) implies
        continued_fraction_convergent_value(coefficients, Nat.2 * n) <
            continued_fraction_convergent_value(
                coefficients, (Nat.2 * n).suc)
} by {
    if positive_continued_fraction_sequence_tail(coefficients) {
        continued_fraction_alternating_sign_double(n)
        continued_fraction_convergent_value_lt_next_of_positive_sign(
            coefficients, Nat.2 * n)
    }
}

/// Each odd-indexed convergent is strictly above the following even-indexed
/// convergent.
theorem continued_fraction_following_even_lt_odd_convergent(
    coefficients: Nat -> Nat, n: Nat
) {
    positive_continued_fraction_sequence_tail(coefficients) implies
        continued_fraction_convergent_value(
                coefficients, (Nat.2 * n).suc.suc) <
            continued_fraction_convergent_value(
                coefficients, (Nat.2 * n).suc)
} by {
    if positive_continued_fraction_sequence_tail(coefficients) {
        continued_fraction_alternating_sign_double_suc(n)
        continued_fraction_convergent_value_gt_next_of_negative_sign(
            coefficients, (Nat.2 * n).suc)
    }
}

/// The cross product of adjacent convergent pairs is the alternating sign of
/// the lower index.
theorem continued_fraction_convergent_sequence_adjacent_identity(
    coefficients: Nat -> Nat, n: Nat
) {
    Int.from_nat(continued_fraction_convergent_sequence(
        coefficients, n.suc).first) *
        Int.from_nat(continued_fraction_convergent_sequence(
            coefficients, n).second) -
    Int.from_nat(continued_fraction_convergent_sequence(
        coefficients, n).first) *
        Int.from_nat(continued_fraction_convergent_sequence(
            coefficients, n.suc).second) =
        alternating_sign[Int](n)
} by {
    continued_fraction_convergent_sequence_first(coefficients, n)
    continued_fraction_convergent_sequence_second(coefficients, n)
    continued_fraction_convergent_sequence_first(coefficients, n.suc)
    continued_fraction_convergent_sequence_second(coefficients, n.suc)
    continued_fraction_adjacent_convergent_determinant_identity(coefficients, n)
}

/// A common summand may be removed from a natural-number upper bound.
theorem continued_fraction_lte_cancel_add_left(
    shared: Nat, left: Nat, right: Nat
) {
    shared + left <= shared + right implies left <= right
} by {
    if shared + left <= shared + right {
        let difference: Nat satisfy {
            shared + left + difference = shared + right
        }
        add_assoc(shared, left, difference)
        shared + (left + difference) = shared + right
        add_cancels_left(shared, left + difference, right)
        left + difference = right
        left <= right
    }
}

/// A strict natural-number inequality remains true as a successor upper
/// bound.
theorem continued_fraction_add_one_lte_of_lt(left: Nat, right: Nat) {
    left < right implies left + Nat.1 <= right
} by {
    if left < right {
        lt_imp_lte_suc(left, right)
        add_one_right(left)
        left + Nat.1 <= right
    }
}

/// A fraction strictly between determinant-one neighbors has denominator at
/// least the sum of the neighboring denominators.
theorem continued_fraction_between_neighbors_denominator_bound(
    left_numerator: Nat, left_denominator: Nat,
    right_numerator: Nat, right_denominator: Nat,
    middle_numerator: Nat, middle_denominator: Nat
) {
    left_numerator * right_denominator + Nat.1 =
            right_numerator * left_denominator and
        left_numerator * middle_denominator <
            middle_numerator * left_denominator and
        middle_numerator * right_denominator <
            right_numerator * middle_denominator implies
        left_denominator + right_denominator <= middle_denominator
} by {
    if left_numerator * right_denominator + Nat.1 =
                right_numerator * left_denominator and
            left_numerator * middle_denominator <
                middle_numerator * left_denominator and
            middle_numerator * right_denominator <
                right_numerator * middle_denominator {
        continued_fraction_add_one_lte_of_lt(
            left_numerator * middle_denominator,
            middle_numerator * left_denominator)
        continued_fraction_add_one_lte_of_lt(
            middle_numerator * right_denominator,
            right_numerator * middle_denominator)

        lte_mul_both(
            right_denominator,
            left_numerator * middle_denominator + Nat.1,
            middle_numerator * left_denominator)
        distrib_left(
            right_denominator,
            left_numerator * middle_denominator,
            Nat.1)
        right_denominator * Nat.1 = right_denominator
        right_denominator * (left_numerator * middle_denominator) =
            left_numerator * middle_denominator * right_denominator
        left_numerator * middle_denominator * right_denominator + right_denominator <= middle_numerator * left_denominator * right_denominator

        lte_mul_both(
            left_denominator,
            middle_numerator * right_denominator + Nat.1,
            right_numerator * middle_denominator)
        distrib_left(
            left_denominator,
            middle_numerator * right_denominator,
            Nat.1)
        left_denominator * Nat.1 = left_denominator
        left_denominator * (middle_numerator * right_denominator) =
            middle_numerator * left_denominator * right_denominator
        middle_numerator * left_denominator * right_denominator + left_denominator <= right_numerator * left_denominator * middle_denominator

        lte_add_right(
            left_denominator,
            left_numerator * middle_denominator * right_denominator +
                right_denominator,
            middle_numerator * left_denominator * right_denominator)
        left_numerator * middle_denominator * right_denominator + right_denominator + left_denominator <= middle_numerator * left_denominator * right_denominator + left_denominator
        lte_trans(
            left_numerator * middle_denominator * right_denominator +
                right_denominator + left_denominator,
            middle_numerator * left_denominator * right_denominator +
                left_denominator,
            right_numerator * left_denominator * middle_denominator)
        left_numerator * middle_denominator * right_denominator + right_denominator + left_denominator <= right_numerator * left_denominator * middle_denominator

        right_numerator * left_denominator =
            left_numerator * right_denominator + Nat.1
        right_numerator * left_denominator * middle_denominator =
            (left_numerator * right_denominator + Nat.1) *
                middle_denominator
        distrib_right(
            left_numerator * right_denominator,
            Nat.1,
            middle_denominator)
        Nat.1 * middle_denominator = middle_denominator
        left_numerator * right_denominator * middle_denominator =
            left_numerator * middle_denominator * right_denominator
        right_numerator * left_denominator * middle_denominator =
            left_numerator * middle_denominator * right_denominator +
                middle_denominator
        add_assoc(
            left_numerator * middle_denominator * right_denominator,
            right_denominator,
            left_denominator)
        left_numerator * middle_denominator * right_denominator + (right_denominator + left_denominator) <= left_numerator * middle_denominator * right_denominator + middle_denominator
        continued_fraction_lte_cancel_add_left(
            left_numerator * middle_denominator * right_denominator,
            right_denominator + left_denominator,
            middle_denominator)
        right_denominator + left_denominator <= middle_denominator
        add_comm(right_denominator, left_denominator)
        left_denominator + right_denominator <= middle_denominator
    }
}

/// At a positive alternating sign, any fraction strictly between adjacent
/// convergents has denominator at least the sum of their denominators.
theorem continued_fraction_between_adjacent_denominator_bound_of_positive_sign(
    coefficients: Nat -> Nat, n: Nat,
    middle_numerator: Nat, middle_denominator: Nat
) {
    alternating_sign[Int](n) = Int.1 and
        continued_fraction_convergent_numerator(coefficients, n) *
                middle_denominator <
            middle_numerator *
                continued_fraction_convergent_denominator(coefficients, n) and
        middle_numerator *
                continued_fraction_convergent_denominator(
                    coefficients, n.suc) <
            continued_fraction_convergent_numerator(coefficients, n.suc) *
                middle_denominator implies
        continued_fraction_convergent_denominator(coefficients, n) +
            continued_fraction_convergent_denominator(coefficients, n.suc) <=
                middle_denominator
} by {
    if alternating_sign[Int](n) = Int.1 and
            continued_fraction_convergent_numerator(coefficients, n) *
                    middle_denominator <
                middle_numerator *
                    continued_fraction_convergent_denominator(
                        coefficients, n) and
            middle_numerator *
                    continued_fraction_convergent_denominator(
                        coefficients, n.suc) <
                continued_fraction_convergent_numerator(
                    coefficients, n.suc) * middle_denominator {
        continued_fraction_adjacent_cross_product_of_positive_sign(
            coefficients, n)
        continued_fraction_between_neighbors_denominator_bound(
            continued_fraction_convergent_numerator(coefficients, n),
            continued_fraction_convergent_denominator(coefficients, n),
            continued_fraction_convergent_numerator(coefficients, n.suc),
            continued_fraction_convergent_denominator(coefficients, n.suc),
            middle_numerator,
            middle_denominator)
    }
}

/// At a negative alternating sign, any fraction strictly between adjacent
/// convergents has denominator at least the sum of their denominators.
theorem continued_fraction_between_adjacent_denominator_bound_of_negative_sign(
    coefficients: Nat -> Nat, n: Nat,
    middle_numerator: Nat, middle_denominator: Nat
) {
    alternating_sign[Int](n) = -Int.1 and
        continued_fraction_convergent_numerator(coefficients, n.suc) *
                middle_denominator <
            middle_numerator *
                continued_fraction_convergent_denominator(
                    coefficients, n.suc) and
        middle_numerator *
                continued_fraction_convergent_denominator(coefficients, n) <
            continued_fraction_convergent_numerator(coefficients, n) *
                middle_denominator implies
        continued_fraction_convergent_denominator(coefficients, n) +
            continued_fraction_convergent_denominator(coefficients, n.suc) <=
                middle_denominator
} by {
    if alternating_sign[Int](n) = -Int.1 and
            continued_fraction_convergent_numerator(coefficients, n.suc) *
                    middle_denominator <
                middle_numerator *
                    continued_fraction_convergent_denominator(
                        coefficients, n.suc) and
            middle_numerator *
                    continued_fraction_convergent_denominator(
                        coefficients, n) <
                continued_fraction_convergent_numerator(coefficients, n) *
                    middle_denominator {
        continued_fraction_adjacent_cross_product_of_negative_sign(
            coefficients, n)
        continued_fraction_between_neighbors_denominator_bound(
            continued_fraction_convergent_numerator(coefficients, n.suc),
            continued_fraction_convergent_denominator(coefficients, n.suc),
            continued_fraction_convergent_numerator(coefficients, n),
            continued_fraction_convergent_denominator(coefficients, n),
            middle_numerator,
            middle_denominator)
        continued_fraction_convergent_denominator(coefficients, n.suc) + continued_fraction_convergent_denominator(coefficients, n) <= middle_denominator
        add_comm(
            continued_fraction_convergent_denominator(coefficients, n.suc),
            continued_fraction_convergent_denominator(coefficients, n))
        continued_fraction_convergent_denominator(coefficients, n) + continued_fraction_convergent_denominator(coefficients, n.suc) <= middle_denominator
    }
}

// ============================================================================
// Section: coprimality of consecutive convergents
// ============================================================================

/// The alternating sign at an index is either one or negative one.
///
/// This duplicates `continued_fraction_alternating_sign_one_or_neg_one` from
/// continued_fraction_approx.ac; it is restated here so that the coprimality
/// results below do not import a file that imports this one.
theorem continued_fraction_alternating_sign_is_one_or_neg_one(n: Nat) {
    alternating_sign[Int](n) = Int.1 or alternating_sign[Int](n) = -Int.1
} by {
    define p(k: Nat) -> Bool {
        alternating_sign[Int](k) = Int.1 or alternating_sign[Int](k) = -Int.1
    }
    alternating_sign_zero[Int]
    alternating_sign[Int](Nat.0) = Int.1
    p(Nat.0)
    forall(k: Nat) {
        if p(k) {
            alternating_sign_suc[Int](k)
            alternating_sign[Int](k.suc) = -alternating_sign[Int](k)
            if alternating_sign[Int](k) = Int.1 {
                alternating_sign[Int](k.suc) = -Int.1
                p(k.suc)
            }
            if alternating_sign[Int](k) = -Int.1 {
                alternating_sign[Int](k.suc) = -(-Int.1)
                --Int.1 = Int.1
                alternating_sign[Int](k.suc) = Int.1
                p(k.suc)
            }
            alternating_sign[Int](k) = Int.1 or alternating_sign[Int](k) = -Int.1
            p(k.suc)
        }
        p(k) implies p(k.suc)
    }
    p(Nat.0) and forall(k: Nat) { p(k) implies p(k.suc) }
    Nat.induction(p)
    forall(k: Nat) { p(k) }
    p(n)
}

/// A common divisor of a convergent numerator and its convergent denominator
/// divides one.
theorem continued_fraction_common_divisor_divides_one(
    coefficients: Nat -> Nat, n: Nat, d: Nat
) {
    d.divides(continued_fraction_convergent_numerator(coefficients, n)) and
        d.divides(continued_fraction_convergent_denominator(coefficients, n))
        implies d.divides(Nat.1)
} by {
    if d.divides(continued_fraction_convergent_numerator(coefficients, n)) and
            d.divides(continued_fraction_convergent_denominator(coefficients, n)) {
        continued_fraction_alternating_sign_is_one_or_neg_one(n)
        if alternating_sign[Int](n) = Int.1 {
            continued_fraction_adjacent_cross_product_of_positive_sign(
                coefficients, n)
            continued_fraction_convergent_numerator(coefficients, n) *
                    continued_fraction_convergent_denominator(
                        coefficients, n.suc) + Nat.1 =
                continued_fraction_convergent_numerator(coefficients, n.suc) *
                    continued_fraction_convergent_denominator(coefficients, n)
            divides_mul(
                continued_fraction_convergent_numerator(coefficients, n),
                continued_fraction_convergent_denominator(coefficients, n.suc),
                d)
            d.divides(continued_fraction_convergent_numerator(coefficients, n) *
                continued_fraction_convergent_denominator(coefficients, n.suc))
            divides_mul(
                continued_fraction_convergent_denominator(coefficients, n),
                continued_fraction_convergent_numerator(coefficients, n.suc),
                d)
            d.divides(continued_fraction_convergent_denominator(coefficients, n) *
                continued_fraction_convergent_numerator(coefficients, n.suc))
            mul_comm(
                continued_fraction_convergent_denominator(coefficients, n),
                continued_fraction_convergent_numerator(coefficients, n.suc))
            continued_fraction_convergent_denominator(coefficients, n) *
                    continued_fraction_convergent_numerator(
                        coefficients, n.suc) =
                continued_fraction_convergent_numerator(coefficients, n.suc) *
                    continued_fraction_convergent_denominator(coefficients, n)
            d.divides(continued_fraction_convergent_numerator(coefficients, n.suc) *
                continued_fraction_convergent_denominator(coefficients, n))
            divides_sub(
                continued_fraction_convergent_numerator(coefficients, n.suc) *
                    continued_fraction_convergent_denominator(coefficients, n),
                continued_fraction_convergent_numerator(coefficients, n) *
                    continued_fraction_convergent_denominator(
                        coefficients, n.suc),
                d)
            d.divides(continued_fraction_convergent_numerator(coefficients, n.suc) *
                    continued_fraction_convergent_denominator(coefficients, n) -
                continued_fraction_convergent_numerator(coefficients, n) *
                    continued_fraction_convergent_denominator(
                        coefficients, n.suc))
            add_imp_sub_left(
                continued_fraction_convergent_numerator(coefficients, n) *
                    continued_fraction_convergent_denominator(
                        coefficients, n.suc),
                Nat.1,
                continued_fraction_convergent_numerator(coefficients, n.suc) *
                    continued_fraction_convergent_denominator(coefficients, n))
            continued_fraction_convergent_numerator(coefficients, n.suc) *
                    continued_fraction_convergent_denominator(coefficients, n) -
                continued_fraction_convergent_numerator(coefficients, n) *
                    continued_fraction_convergent_denominator(
                        coefficients, n.suc) =
                Nat.1
            d.divides(Nat.1)
        }
        if alternating_sign[Int](n) = -Int.1 {
            continued_fraction_adjacent_cross_product_of_negative_sign(
                coefficients, n)
            continued_fraction_convergent_numerator(coefficients, n.suc) *
                    continued_fraction_convergent_denominator(coefficients, n) +
                    Nat.1 =
                continued_fraction_convergent_numerator(coefficients, n) *
                    continued_fraction_convergent_denominator(
                        coefficients, n.suc)
            divides_mul(
                continued_fraction_convergent_numerator(coefficients, n),
                continued_fraction_convergent_denominator(coefficients, n.suc),
                d)
            d.divides(continued_fraction_convergent_numerator(coefficients, n) *
                continued_fraction_convergent_denominator(coefficients, n.suc))
            divides_mul(
                continued_fraction_convergent_denominator(coefficients, n),
                continued_fraction_convergent_numerator(coefficients, n.suc),
                d)
            d.divides(continued_fraction_convergent_denominator(coefficients, n) *
                continued_fraction_convergent_numerator(coefficients, n.suc))
            mul_comm(
                continued_fraction_convergent_denominator(coefficients, n),
                continued_fraction_convergent_numerator(coefficients, n.suc))
            continued_fraction_convergent_denominator(coefficients, n) *
                    continued_fraction_convergent_numerator(
                        coefficients, n.suc) =
                continued_fraction_convergent_numerator(coefficients, n.suc) *
                    continued_fraction_convergent_denominator(coefficients, n)
            d.divides(continued_fraction_convergent_numerator(coefficients, n.suc) *
                continued_fraction_convergent_denominator(coefficients, n))
            divides_sub(
                continued_fraction_convergent_numerator(coefficients, n) *
                    continued_fraction_convergent_denominator(
                        coefficients, n.suc),
                continued_fraction_convergent_numerator(coefficients, n.suc) *
                    continued_fraction_convergent_denominator(coefficients, n),
                d)
            d.divides(continued_fraction_convergent_numerator(coefficients, n) *
                    continued_fraction_convergent_denominator(
                        coefficients, n.suc) -
                continued_fraction_convergent_numerator(coefficients, n.suc) *
                    continued_fraction_convergent_denominator(coefficients, n))
            add_imp_sub_left(
                continued_fraction_convergent_numerator(coefficients, n.suc) *
                    continued_fraction_convergent_denominator(coefficients, n),
                Nat.1,
                continued_fraction_convergent_numerator(coefficients, n) *
                    continued_fraction_convergent_denominator(
                        coefficients, n.suc))
            continued_fraction_convergent_numerator(coefficients, n) *
                    continued_fraction_convergent_denominator(
                        coefficients, n.suc) -
                continued_fraction_convergent_numerator(coefficients, n.suc) *
                    continued_fraction_convergent_denominator(coefficients, n) =
                Nat.1
            d.divides(Nat.1)
        }
        d.divides(Nat.1)
    }
}

/// Consecutive convergent numerators and denominators are coprime: the
/// greatest common divisor of `p_n` and `q_n` is one.
theorem continued_fraction_convergent_gcd_one(
    coefficients: Nat -> Nat, n: Nat
) {
    continued_fraction_convergent_numerator(coefficients, n).gcd(
        continued_fraction_convergent_denominator(coefficients, n)) = Nat.1
} by {
    gcd_divides_left(
        continued_fraction_convergent_numerator(coefficients, n),
        continued_fraction_convergent_denominator(coefficients, n))
    continued_fraction_convergent_numerator(coefficients, n).gcd(continued_fraction_convergent_denominator(coefficients, n)).divides(continued_fraction_convergent_numerator(coefficients, n))
    gcd_divides_right(
        continued_fraction_convergent_numerator(coefficients, n),
        continued_fraction_convergent_denominator(coefficients, n))
    continued_fraction_convergent_numerator(coefficients, n).gcd(continued_fraction_convergent_denominator(coefficients, n)).divides(continued_fraction_convergent_denominator(coefficients, n))
    continued_fraction_common_divisor_divides_one(
        coefficients, n,
        continued_fraction_convergent_numerator(coefficients, n).gcd(
            continued_fraction_convergent_denominator(coefficients, n)))
    continued_fraction_convergent_numerator(coefficients, n).gcd(continued_fraction_convergent_denominator(coefficients, n)).divides(Nat.1)
    nat_divides_one_imp_one(
        continued_fraction_convergent_numerator(coefficients, n).gcd(
            continued_fraction_convergent_denominator(coefficients, n)))
    continued_fraction_convergent_numerator(coefficients, n).gcd(
        continued_fraction_convergent_denominator(coefficients, n)) = Nat.1
}

/// Consecutive convergent numerators and denominators are coprime.
theorem continued_fraction_convergent_coprime(
    coefficients: Nat -> Nat, n: Nat
) {
    continued_fraction_convergent_numerator(coefficients, n).coprime(
        continued_fraction_convergent_denominator(coefficients, n))
} by {
    continued_fraction_convergent_gcd_one(coefficients, n)
    continued_fraction_convergent_numerator(coefficients, n).gcd(
        continued_fraction_convergent_denominator(coefficients, n)) = Nat.1
}
