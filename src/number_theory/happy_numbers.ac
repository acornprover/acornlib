from nat import Nat, div_lt, zero_or_suc, suc_sub_one, strong_induction, true_below,
    true_below_apply, only_zero_lte_zero, lt_imp_lte_suc, lte_cancel_suc,
    mul_to_zero, lte_trans, lte_mul_both, lt_add_suc, add_zero_left,
    add_zero_right, add_comm, add_assoc, mul_comm, mul_zero_left,
    mul_one_right, small_mod, div_of_decomp, mod_of_decomp, read_add_read,
    read_read_carry, read_add_single, nat_mul_2_2, nat_mul_3_3,
    nat_mul_4_4, nat_mul_5_5, nat_mul_6_6, nat_mul_7_7, nat_mul_8_8,
    nat_mul_9_9, nine_plus_one, five_plus_four
numerals Nat

/// The remainder of a base multiple plus a small digit is that digit.
theorem mod_base_mul_add_digit(p: Nat, n: Nat, d: Nat) {
    d < p implies (p * n + d).mod(p) = d
} by {
    if d < p {
        p * n = n * p
        p * n + d = n * p + d
        mod_of_decomp(n, d, p)
        (n * p + d).mod(p) = d
        (p * n + d).mod(p) = d
    }
}

/// The quotient of a base multiple plus a small digit is the multiplier.
theorem div_base_mul_add_digit(p: Nat, n: Nat, d: Nat) {
    d < p implies (p * n + d).div(p) = n
} by {
    if d < p {
        p * n = n * p
        p * n + d = n * p + d
        div_of_decomp(n, d, p)
        (n * p + d).div(p) = n
        (p * n + d).div(p) = n
    }
}

/// If a sum is zero, both summands are zero.
theorem nat_add_to_zero(a: Nat, b: Nat) {
    a + b = Nat.0 implies a = Nat.0 and b = Nat.0
} by {
    if a + b = Nat.0 {
        a = Nat.0
        b = Nat.0
    }
}

/// The digit zero is less than ten.
theorem digit_zero_lt_ten {
    Nat.0 < Nat.10
}

/// The digit one is less than ten.
theorem digit_one_lt_ten {
    Nat.1 < Nat.10
} by {
    lt_add_suc(Nat.1, Nat.8)
    Nat.1 < Nat.1 + Nat.9
    Nat.1 + Nat.9 = Nat.10
    Nat.1 < Nat.10
}

/// The digit two is less than ten.
theorem digit_two_lt_ten {
    Nat.2 < Nat.10
} by {
    lt_add_suc(Nat.2, Nat.7)
    Nat.2 < Nat.2 + Nat.8
    Nat.2 + Nat.8 = Nat.10
    Nat.2 < Nat.10
}

/// The digit three is less than ten.
theorem digit_three_lt_ten {
    Nat.3 < Nat.10
} by {
    lt_add_suc(Nat.3, Nat.6)
    Nat.3 < Nat.3 + Nat.7
    Nat.7 + Nat.3 = Nat.10
    Nat.3 + Nat.7 = Nat.7 + Nat.3
    Nat.3 + Nat.7 = Nat.10
    Nat.3 < Nat.10
}

/// The digit four is less than ten.
theorem digit_four_lt_ten {
    Nat.4 < Nat.10
} by {
    lt_add_suc(Nat.4, Nat.5)
    Nat.4 < Nat.4 + Nat.6
    Nat.6 + Nat.4 = Nat.10
    Nat.4 + Nat.6 = Nat.6 + Nat.4
    Nat.4 + Nat.6 = Nat.10
    Nat.4 < Nat.10
}

/// The digit five is less than ten.
theorem digit_five_lt_ten {
    Nat.5 < Nat.10
} by {
    lt_add_suc(Nat.5, Nat.4)
    Nat.5 < Nat.5 + Nat.5
    Nat.5 + Nat.5 = Nat.10
    Nat.5 < Nat.10
}

/// The digit six is less than ten.
theorem digit_six_lt_ten {
    Nat.6 < Nat.10
} by {
    lt_add_suc(Nat.6, Nat.3)
    Nat.6 < Nat.6 + Nat.4
    Nat.6 + Nat.4 = Nat.10
    Nat.6 < Nat.10
}

/// The digit seven is less than ten.
theorem digit_seven_lt_ten {
    Nat.7 < Nat.10
} by {
    lt_add_suc(Nat.7, Nat.2)
    Nat.7 < Nat.7 + Nat.3
    Nat.7 + Nat.3 = Nat.10
    Nat.7 < Nat.10
}

/// The digit eight is less than ten.
theorem digit_eight_lt_ten {
    Nat.8 < Nat.10
} by {
    lt_add_suc(Nat.8, Nat.1)
    Nat.8 < Nat.8 + Nat.2
    Nat.8 + Nat.2 = Nat.10
    Nat.8 < Nat.10
}

/// The digit nine is less than ten.
theorem digit_nine_lt_ten {
    Nat.9 < Nat.10
} by {
    lt_add_suc(Nat.9, Nat.0)
    Nat.9 < Nat.9 + Nat.1
    nine_plus_one
    Nat.9 + Nat.1 = Nat.10
    Nat.9 < Nat.10
}

/// `8 + 4 = 12`.
theorem nat_add_8_4 {
    Nat.8 + Nat.4 = Nat.12
} by {
    Nat.4 = Nat.2 + Nat.2
    Nat.8 + Nat.4 = Nat.8 + (Nat.2 + Nat.2)
    add_assoc(Nat.8, Nat.2, Nat.2)
    Nat.8 + Nat.4 = (Nat.8 + Nat.2) + Nat.2
    Nat.8 + Nat.2 = Nat.10
    Nat.8 + Nat.4 = Nat.10 + Nat.2
    Nat.10 + Nat.2 = Nat.12
    Nat.8 + Nat.4 = Nat.12
}

/// `5 + 7 = 12`.
theorem nat_add_5_7 {
    Nat.5 + Nat.7 = Nat.12
} by {
    Nat.7 = Nat.5 + Nat.2
    Nat.5 + Nat.7 = Nat.5 + (Nat.5 + Nat.2)
    add_assoc(Nat.5, Nat.5, Nat.2)
    Nat.5 + Nat.7 = (Nat.5 + Nat.5) + Nat.2
    Nat.5 + Nat.5 = Nat.10
    Nat.5 + Nat.7 = Nat.10 + Nat.2
    Nat.10 + Nat.2 = Nat.12
    Nat.5 + Nat.7 = Nat.12
}

/// `7 + 5 = 12`.
theorem nat_add_7_5 {
    Nat.7 + Nat.5 = Nat.12
} by {
    nat_add_5_7
    Nat.5 + Nat.7 = Nat.12
    add_comm(Nat.7, Nat.5)
    Nat.7 + Nat.5 = Nat.5 + Nat.7
    Nat.7 + Nat.5 = Nat.12
}

/// `6 + 8 = 14`.
theorem nat_add_6_8 {
    Nat.6 + Nat.8 = Nat.14
} by {
    Nat.8 = Nat.4 + Nat.4
    Nat.6 + Nat.8 = Nat.6 + (Nat.4 + Nat.4)
    add_assoc(Nat.6, Nat.4, Nat.4)
    Nat.6 + Nat.8 = (Nat.6 + Nat.4) + Nat.4
    Nat.6 + Nat.4 = Nat.10
    Nat.6 + Nat.8 = Nat.10 + Nat.4
    Nat.10 + Nat.4 = Nat.14
    Nat.6 + Nat.8 = Nat.14
}

/// `81 + 64 = 145`.
theorem nat_add_81_64 {
    Nat.81 + Nat.64 = Nat.145
} by {
    Nat.81 = Nat.8.read(Nat.1)
    Nat.64 = Nat.6.read(Nat.4)
    read_add_read(Nat.8, Nat.1, Nat.6, Nat.4)
    Nat.8.read(Nat.1) + Nat.6.read(Nat.4) =
        (Nat.8 + Nat.6).read(Nat.1 + Nat.4)
    Nat.81 + Nat.64 = (Nat.8 + Nat.6).read(Nat.1 + Nat.4)
    nat_add_6_8
    Nat.6 + Nat.8 = Nat.14
    add_comm(Nat.8, Nat.6)
    Nat.8 + Nat.6 = Nat.6 + Nat.8
    Nat.8 + Nat.6 = Nat.14
    Nat.1 + Nat.4 = Nat.5
    Nat.81 + Nat.64 = Nat.14.read(Nat.5)
    Nat.14.read(Nat.5) = Nat.145
    Nat.81 + Nat.64 = Nat.145
}

/// `81 + 49 = 130`.
theorem nat_add_81_49 {
    Nat.81 + Nat.49 = Nat.130
} by {
    Nat.81 = Nat.8.read(Nat.1)
    Nat.49 = Nat.4.read(Nat.9)
    read_add_read(Nat.8, Nat.1, Nat.4, Nat.9)
    Nat.8.read(Nat.1) + Nat.4.read(Nat.9) =
        (Nat.8 + Nat.4).read(Nat.1 + Nat.9)
    Nat.81 + Nat.49 = (Nat.8 + Nat.4).read(Nat.1 + Nat.9)
    nat_add_8_4
    Nat.8 + Nat.4 = Nat.12
    Nat.1 + Nat.9 = Nat.10
    Nat.81 + Nat.49 = Nat.12.read(Nat.10)
    Nat.10 = Nat.10 * Nat.1 + Nat.0
    Nat.12.read(Nat.10) = Nat.12.read(Nat.10 * Nat.1 + Nat.0)
    read_read_carry(Nat.12, Nat.1, Nat.0)
    Nat.12.read(Nat.10 * Nat.1 + Nat.0) = (Nat.12 + Nat.1).read(Nat.0)
    Nat.12 + Nat.1 = Nat.13
    Nat.81 + Nat.49 = Nat.13.read(Nat.0)
    Nat.13.read(Nat.0) = Nat.130
    Nat.81 + Nat.49 = Nat.130
}

/// `9 + 49 = 58`.
theorem nat_add_9_49 {
    Nat.9 + Nat.49 = Nat.58
} by {
    Nat.9 = Nat.0.read(Nat.9)
    Nat.49 = Nat.4.read(Nat.9)
    read_add_read(Nat.0, Nat.9, Nat.4, Nat.9)
    Nat.0.read(Nat.9) + Nat.4.read(Nat.9) =
        (Nat.0 + Nat.4).read(Nat.9 + Nat.9)
    Nat.9 + Nat.49 = (Nat.0 + Nat.4).read(Nat.9 + Nat.9)
    Nat.0 + Nat.4 = Nat.4
    Nat.9 + Nat.9 = Nat.18
    Nat.9 + Nat.49 = Nat.4.read(Nat.18)
    Nat.18 = Nat.10 * Nat.1 + Nat.8
    Nat.4.read(Nat.18) = Nat.4.read(Nat.10 * Nat.1 + Nat.8)
    read_read_carry(Nat.4, Nat.1, Nat.8)
    Nat.4.read(Nat.10 * Nat.1 + Nat.8) = (Nat.4 + Nat.1).read(Nat.8)
    Nat.4 + Nat.1 = Nat.5
    Nat.9 + Nat.49 = Nat.5.read(Nat.8)
    Nat.5.read(Nat.8) = Nat.58
    Nat.9 + Nat.49 = Nat.58
}

/// `25 + 64 = 89`.
theorem nat_add_25_64 {
    Nat.25 + Nat.64 = Nat.89
} by {
    Nat.25 = Nat.2.read(Nat.5)
    Nat.64 = Nat.6.read(Nat.4)
    read_add_read(Nat.2, Nat.5, Nat.6, Nat.4)
    Nat.2.read(Nat.5) + Nat.6.read(Nat.4) =
        (Nat.2 + Nat.6).read(Nat.5 + Nat.4)
    Nat.25 + Nat.64 = (Nat.2 + Nat.6).read(Nat.5 + Nat.4)
    Nat.2 + Nat.6 = Nat.8
    five_plus_four
    Nat.5 + Nat.4 = Nat.9
    Nat.25 + Nat.64 = Nat.8.read(Nat.9)
    Nat.8.read(Nat.9) = Nat.89
    Nat.25 + Nat.64 = Nat.89
}

/// `17 + 25 = 42`.
theorem nat_add_17_25 {
    Nat.17 + Nat.25 = Nat.42
} by {
    Nat.17 = Nat.1.read(Nat.7)
    Nat.25 = Nat.2.read(Nat.5)
    read_add_read(Nat.1, Nat.7, Nat.2, Nat.5)
    Nat.1.read(Nat.7) + Nat.2.read(Nat.5) =
        (Nat.1 + Nat.2).read(Nat.7 + Nat.5)
    Nat.17 + Nat.25 = (Nat.1 + Nat.2).read(Nat.7 + Nat.5)
    Nat.1 + Nat.2 = Nat.3
    nat_add_7_5
    Nat.7 + Nat.5 = Nat.12
    Nat.17 + Nat.25 = Nat.3.read(Nat.12)
    Nat.12 = Nat.10 * Nat.1 + Nat.2
    Nat.3.read(Nat.12) = Nat.3.read(Nat.10 * Nat.1 + Nat.2)
    read_read_carry(Nat.3, Nat.1, Nat.2)
    Nat.3.read(Nat.10 * Nat.1 + Nat.2) = (Nat.3 + Nat.1).read(Nat.2)
    Nat.3 + Nat.1 = Nat.4
    Nat.17 + Nat.25 = Nat.4.read(Nat.2)
    Nat.4.read(Nat.2) = Nat.42
    Nat.17 + Nat.25 = Nat.42
}

/// `16 + 4 = 20`.
theorem nat_add_16_4 {
    Nat.16 + Nat.4 = Nat.20
} by {
    Nat.16 = Nat.1.read(Nat.6)
    read_add_single(Nat.1, Nat.6, Nat.4)
    Nat.1.read(Nat.6) + Nat.4 = Nat.1.read(Nat.6 + Nat.4)
    Nat.16 + Nat.4 = Nat.1.read(Nat.6 + Nat.4)
    Nat.6 + Nat.4 = Nat.10
    Nat.16 + Nat.4 = Nat.1.read(Nat.10)
    Nat.10 = Nat.10 * Nat.1 + Nat.0
    Nat.1.read(Nat.10) = Nat.1.read(Nat.10 * Nat.1 + Nat.0)
    read_read_carry(Nat.1, Nat.1, Nat.0)
    Nat.1.read(Nat.10 * Nat.1 + Nat.0) = (Nat.1 + Nat.1).read(Nat.0)
    Nat.1 + Nat.1 = Nat.2
    Nat.16 + Nat.4 = Nat.2.read(Nat.0)
    Nat.2.read(Nat.0) = Nat.20
    Nat.16 + Nat.4 = Nat.20
}

/// The sum of the squares of the base-`p` digits of `n`, computed with an
/// explicit recursion budget `fuel`. Any `fuel >= n` gives the true value.
define happy_fuel(p: Nat, n: Nat, fuel: Nat) -> Nat {
    match fuel {
        Nat.zero {
            Nat.0
        }
        Nat.suc(k) {
            if n = Nat.0 {
                Nat.0
            } else {
                n.mod(p) * n.mod(p) + happy_fuel(p, n.div(p), k)
            }
        }
    }
}

/// The sum of the squares of the base-`p` digits of `n` (the happy function in
/// base `p`).
define happy_sum(p: Nat, n: Nat) -> Nat {
    happy_fuel(p, n, n)
}

/// True if a recursion budget is large enough to give the true digit-square sum.
define happy_fuel_large_case(p: Nat, n: Nat, fuel: Nat) -> Bool {
    Nat.1 < p and n <= fuel implies happy_fuel(p, n, fuel) = happy_sum(p, n)
}

/// The digit-square sum of zero vanishes, for any recursion budget.
theorem happy_fuel_at_zero(p: Nat, fuel: Nat) {
    happy_fuel(p, Nat.0, fuel) = Nat.0
} by {
    zero_or_suc(fuel)
}

/// One step of the digit-square recursion peels off the lowest digit.
theorem happy_fuel_step(p: Nat, n: Nat, k: Nat) {
    n != Nat.0 implies
        happy_fuel(p, n, k.suc) = n.mod(p) * n.mod(p) + happy_fuel(p, n.div(p), k)
}

/// The digit-square sum of a nonzero number splits into its final digit and
/// quotient.
theorem happy_sum_step(p: Nat, n: Nat) {
    n != Nat.0 implies
        happy_sum(p, n) =
            n.mod(p) * n.mod(p) + happy_fuel(p, n.div(p), n - Nat.1)
} by {
    if n != Nat.0 {
        zero_or_suc(n)
        if n = Nat.0 {
            false
        }
        let k: Nat satisfy { n = k.suc }
        suc_sub_one(k)
        happy_fuel_step(p, n, k)
        happy_sum(p, n) =
            n.mod(p) * n.mod(p) + happy_fuel(p, n.div(p), n - Nat.1)
    }
}

/// Oversized recursion budgets give the same digit-square sum.
theorem happy_fuel_large(p: Nat, n: Nat, fuel: Nat) {
    Nat.1 < p and n <= fuel implies happy_fuel(p, n, fuel) = happy_sum(p, n)
} by {
    let f: Nat -> Bool = function(m: Nat) {
        forall(fuel2: Nat) {
            happy_fuel_large_case(p, m, fuel2)
        }
    }
    strong_induction(f)
    forall(m: Nat) {
        if true_below(f, m) {
            forall(fuel2: Nat) {
                if Nat.1 < p and m <= fuel2 {
                    if m = Nat.0 {
                        happy_fuel_at_zero(p, fuel2)
                        happy_fuel_at_zero(p, m)
                        happy_fuel(p, m, fuel2) = happy_sum(p, m)
                    }
                    if m != Nat.0 {
                        zero_or_suc(m)
                        let k: Nat satisfy { m = k.suc }
                        zero_or_suc(fuel2)
                        if fuel2 = Nat.0 {
                            only_zero_lte_zero(m)
                            false
                        }
                        let h: Nat satisfy { fuel2 = h.suc }
                        let q: Nat = m.div(p)
                        div_lt(m, p)
                        q < m
                        lt_imp_lte_suc(q, m)
                        lte_cancel_suc(q, k)
                        lte_cancel_suc(k, h)
                        k <= h
                        lte_trans(q, k, h)
                        suc_sub_one(k)
                        true_below_apply(f, m, q)
                        happy_fuel_large_case(p, q, h)
                        Nat.1 < p and q <= h
                        happy_fuel(p, q, h) = happy_sum(p, q)
                        happy_fuel_large_case(p, q, k)
                        Nat.1 < p and q <= k
                        happy_fuel(p, q, k) = happy_sum(p, q)
                        happy_fuel_step(p, m, h)
                        happy_fuel_step(p, m, k)
                        happy_fuel(p, m, fuel2) = happy_sum(p, m)
                    }
                    happy_fuel_large_case(p, m, fuel2)
                }
            }
            forall(fuel2: Nat) {
                happy_fuel_large_case(p, m, fuel2)
            }
            f(m)
        }
    }
    f(n)
    happy_fuel_large_case(p, n, fuel)
}

/// The digit-square recurrence for a nonzero number in a base greater than one.
theorem happy_sum_recurrence(p: Nat, n: Nat) {
    Nat.1 < p and n != Nat.0 implies
        happy_sum(p, n) = n.mod(p) * n.mod(p) + happy_sum(p, n.div(p))
} by {
    if Nat.1 < p and n != Nat.0 {
        zero_or_suc(n)
        if n = Nat.0 {
            false
        }
        let k: Nat satisfy { n = k.suc }
        let q: Nat = n.div(p)
        div_lt(n, p)
        q < n
        lt_imp_lte_suc(q, n)
        lte_cancel_suc(q, k)
        q <= k
        suc_sub_one(k)
        happy_sum_step(p, n)
        happy_fuel_large(p, q, n - Nat.1)
        n.mod(p) * n.mod(p) + happy_fuel(p, q, n - Nat.1) =
            n.mod(p) * n.mod(p) + happy_sum(p, q)
        happy_sum(p, n) = n.mod(p) * n.mod(p) + happy_sum(p, n.div(p))
    }
}

/// The base-`p` digit-square sum of zero is zero.
theorem happy_sum_zero(p: Nat) {
    happy_sum(p, Nat.0) = Nat.0
} by {
    happy_fuel_at_zero(p, Nat.0)
}

/// Numbers below a nonzero base greater than one have digit-square sum equal to
/// their own square.
theorem happy_sum_small(p: Nat, n: Nat) {
    Nat.1 < p and n < p implies happy_sum(p, n) = n * n
} by {
    if Nat.1 < p and n < p {
        if n = Nat.0 {
            happy_sum_zero(p)
            happy_sum(p, Nat.0) = Nat.0
            happy_sum(p, n) = happy_sum(p, Nat.0)
            happy_sum(p, n) = Nat.0
            n * n = Nat.0 * Nat.0
            Nat.0 * Nat.0 = Nat.0
            n * n = Nat.0
            happy_sum(p, n) = n * n
        }
        if n != Nat.0 {
            happy_sum_recurrence(p, n)
            happy_sum(p, n) = n.mod(p) * n.mod(p) + happy_sum(p, n.div(p))
            small_mod(n, p)
            n.mod(p) = n
            n.mod(p) * n.mod(p) = n * n
            div_of_decomp(Nat.0, n, p)
            mul_zero_left(p)
            add_zero_left(n)
            n.div(p) = Nat.0
            happy_sum_zero(p)
            happy_sum(p, n.div(p)) = happy_sum(p, Nat.0)
            happy_sum(p, n.div(p)) = Nat.0
            happy_sum(p, n) = n * n + Nat.0
            add_zero_right(n * n)
            n * n + Nat.0 = n * n
            happy_sum(p, n) = n * n
        }
        happy_sum(p, n) = n * n
    }
}

/// Appending a digit `d` below a base greater than one adds the square of `d`
/// to the digit-square sum.
theorem happy_sum_base_mul_add_digit(p: Nat, n: Nat, d: Nat) {
    d < p and Nat.1 < p implies
        happy_sum(p, p * n + d) = happy_sum(p, n) + d * d
} by {
    if d < p and Nat.1 < p {
        mod_base_mul_add_digit(p, n, d)
        (p * n + d).mod(p) = d
        div_base_mul_add_digit(p, n, d)
        (p * n + d).div(p) = n
        if p * n + d = Nat.0 {
            nat_add_to_zero(p * n, d)
            p * n = Nat.0
            d = Nat.0
            p != Nat.0
            mul_to_zero(p, n)
            n = Nat.0
            happy_sum_zero(p)
            happy_sum(p, p * n + d) = Nat.0
            happy_sum(p, n) = Nat.0
            happy_sum(p, n) + d * d = Nat.0
            happy_sum(p, p * n + d) = happy_sum(p, n) + d * d
        }
        if p * n + d != Nat.0 {
            happy_sum_recurrence(p, p * n + d)
            happy_sum(p, p * n + d) =
                (p * n + d).mod(p) * (p * n + d).mod(p) +
                    happy_sum(p, (p * n + d).div(p))
            happy_sum(p, p * n + d) = d * d + happy_sum(p, n)
            d * d + happy_sum(p, n) = happy_sum(p, n) + d * d
            happy_sum(p, p * n + d) = happy_sum(p, n) + d * d
        }
        happy_sum(p, p * n + d) = happy_sum(p, n) + d * d
    }
}

/// The sum of the squares of the digits of `1` is `1`.
theorem happy_sum_1 {
    happy_sum(Nat.10, Nat.1) = Nat.1
} by {
    digit_one_lt_ten
    Nat.1 < Nat.10
    happy_sum_small(Nat.10, Nat.1)
    happy_sum(Nat.10, Nat.1) = Nat.1 * Nat.1
    mul_one_right(Nat.1)
    Nat.1 * Nat.1 = Nat.1
    happy_sum(Nat.10, Nat.1) = Nat.1
}

/// The sum of the squares of the digits of `2` is `4`.
theorem happy_sum_2 {
    happy_sum(Nat.10, Nat.2) = Nat.4
} by {
    digit_one_lt_ten
    Nat.1 < Nat.10
    digit_two_lt_ten
    Nat.2 < Nat.10
    happy_sum_small(Nat.10, Nat.2)
    happy_sum(Nat.10, Nat.2) = Nat.2 * Nat.2
    nat_mul_2_2
    Nat.2 * Nat.2 = Nat.4
    happy_sum(Nat.10, Nat.2) = Nat.4
}

/// The sum of the squares of the digits of `3` is `9`.
theorem happy_sum_3 {
    happy_sum(Nat.10, Nat.3) = Nat.9
} by {
    digit_one_lt_ten
    Nat.1 < Nat.10
    digit_three_lt_ten
    Nat.3 < Nat.10
    happy_sum_small(Nat.10, Nat.3)
    happy_sum(Nat.10, Nat.3) = Nat.3 * Nat.3
    nat_mul_3_3
    Nat.3 * Nat.3 = Nat.9
    happy_sum(Nat.10, Nat.3) = Nat.9
}

/// The sum of the squares of the digits of `4` is `16`.
theorem happy_sum_4 {
    happy_sum(Nat.10, Nat.4) = Nat.16
} by {
    digit_one_lt_ten
    Nat.1 < Nat.10
    digit_four_lt_ten
    Nat.4 < Nat.10
    happy_sum_small(Nat.10, Nat.4)
    happy_sum(Nat.10, Nat.4) = Nat.4 * Nat.4
    nat_mul_4_4
    Nat.4 * Nat.4 = Nat.16
    happy_sum(Nat.10, Nat.4) = Nat.16
}

/// The sum of the squares of the digits of `7` is `49`.
theorem happy_sum_7 {
    happy_sum(Nat.10, Nat.7) = Nat.49
} by {
    digit_one_lt_ten
    Nat.1 < Nat.10
    digit_seven_lt_ten
    Nat.7 < Nat.10
    happy_sum_small(Nat.10, Nat.7)
    happy_sum(Nat.10, Nat.7) = Nat.7 * Nat.7
    nat_mul_7_7
    Nat.7 * Nat.7 = Nat.49
    happy_sum(Nat.10, Nat.7) = Nat.49
}

/// The sum of the squares of the digits of `9` is `81`.
theorem happy_sum_9 {
    happy_sum(Nat.10, Nat.9) = Nat.81
} by {
    digit_one_lt_ten
    Nat.1 < Nat.10
    digit_nine_lt_ten
    Nat.9 < Nat.10
    happy_sum_small(Nat.10, Nat.9)
    happy_sum(Nat.10, Nat.9) = Nat.9 * Nat.9
    nat_mul_9_9
    Nat.9 * Nat.9 = Nat.81
    happy_sum(Nat.10, Nat.9) = Nat.81
}

/// The sum of the squares of the digits of `10` is `1`.
theorem happy_sum_10 {
    happy_sum(Nat.10, Nat.10) = Nat.1
} by {
    digit_zero_lt_ten
    Nat.0 < Nat.10
    digit_one_lt_ten
    Nat.1 < Nat.10
    happy_sum_base_mul_add_digit(Nat.10, Nat.1, Nat.0)
    happy_sum(Nat.10, Nat.10 * Nat.1 + Nat.0) =
        happy_sum(Nat.10, Nat.1) + Nat.0 * Nat.0
    happy_sum(Nat.10, Nat.10) = happy_sum(Nat.10, Nat.1) + Nat.0 * Nat.0
    happy_sum_1
    happy_sum(Nat.10, Nat.1) = Nat.1
    Nat.0 * Nat.0 = Nat.0
    happy_sum(Nat.10, Nat.10) = Nat.1 + Nat.0
    add_zero_right(Nat.1)
    Nat.1 + Nat.0 = Nat.1
    happy_sum(Nat.10, Nat.10) = Nat.1
}

/// The sum of the squares of the digits of `13` is `10`.
theorem happy_sum_13 {
    happy_sum(Nat.10, Nat.13) = Nat.10
} by {
    digit_three_lt_ten
    Nat.3 < Nat.10
    digit_one_lt_ten
    Nat.1 < Nat.10
    happy_sum_base_mul_add_digit(Nat.10, Nat.1, Nat.3)
    happy_sum(Nat.10, Nat.10 * Nat.1 + Nat.3) =
        happy_sum(Nat.10, Nat.1) + Nat.3 * Nat.3
    happy_sum(Nat.10, Nat.13) = happy_sum(Nat.10, Nat.1) + Nat.3 * Nat.3
    happy_sum_1
    happy_sum(Nat.10, Nat.1) = Nat.1
    nat_mul_3_3
    Nat.3 * Nat.3 = Nat.9
    happy_sum(Nat.10, Nat.13) = Nat.1 + Nat.9
    Nat.1 + Nat.9 = Nat.10
    happy_sum(Nat.10, Nat.13) = Nat.10
}

/// The sum of the squares of the digits of `5` is `25`.
theorem happy_sum_5 {
    happy_sum(Nat.10, Nat.5) = Nat.25
} by {
    digit_one_lt_ten
    Nat.1 < Nat.10
    digit_five_lt_ten
    Nat.5 < Nat.10
    happy_sum_small(Nat.10, Nat.5)
    happy_sum(Nat.10, Nat.5) = Nat.5 * Nat.5
    nat_mul_5_5
    Nat.5 * Nat.5 = Nat.25
    happy_sum(Nat.10, Nat.5) = Nat.25
}

/// The sum of the squares of the digits of `8` is `64`.
theorem happy_sum_8 {
    happy_sum(Nat.10, Nat.8) = Nat.64
} by {
    digit_one_lt_ten
    Nat.1 < Nat.10
    digit_eight_lt_ten
    Nat.8 < Nat.10
    happy_sum_small(Nat.10, Nat.8)
    happy_sum(Nat.10, Nat.8) = Nat.8 * Nat.8
    nat_mul_8_8
    Nat.8 * Nat.8 = Nat.64
    happy_sum(Nat.10, Nat.8) = Nat.64
}

/// The sum of the squares of the digits of `14` is `17`.
theorem happy_sum_14 {
    happy_sum(Nat.10, Nat.14) = Nat.17
} by {
    digit_four_lt_ten
    Nat.4 < Nat.10
    digit_one_lt_ten
    Nat.1 < Nat.10
    happy_sum_base_mul_add_digit(Nat.10, Nat.1, Nat.4)
    happy_sum(Nat.10, Nat.10 * Nat.1 + Nat.4) =
        happy_sum(Nat.10, Nat.1) + Nat.4 * Nat.4
    happy_sum(Nat.10, Nat.14) = happy_sum(Nat.10, Nat.1) + Nat.4 * Nat.4
    happy_sum_1
    happy_sum(Nat.10, Nat.1) = Nat.1
    nat_mul_4_4
    Nat.4 * Nat.4 = Nat.16
    happy_sum(Nat.10, Nat.14) = Nat.1 + Nat.16
    Nat.1 + Nat.16 = Nat.17
    happy_sum(Nat.10, Nat.14) = Nat.17
}

/// The sum of the squares of the digits of `16` is `37`.
theorem happy_sum_16 {
    happy_sum(Nat.10, Nat.16) = Nat.37
} by {
    digit_six_lt_ten
    Nat.6 < Nat.10
    digit_one_lt_ten
    Nat.1 < Nat.10
    happy_sum_base_mul_add_digit(Nat.10, Nat.1, Nat.6)
    happy_sum(Nat.10, Nat.10 * Nat.1 + Nat.6) =
        happy_sum(Nat.10, Nat.1) + Nat.6 * Nat.6
    happy_sum(Nat.10, Nat.16) = happy_sum(Nat.10, Nat.1) + Nat.6 * Nat.6
    happy_sum_1
    happy_sum(Nat.10, Nat.1) = Nat.1
    nat_mul_6_6
    Nat.6 * Nat.6 = Nat.36
    happy_sum(Nat.10, Nat.16) = Nat.1 + Nat.36
    Nat.1 + Nat.36 = Nat.37
    happy_sum(Nat.10, Nat.16) = Nat.37
}

/// The sum of the squares of the digits of `20` is `4`.
theorem happy_sum_20 {
    happy_sum(Nat.10, Nat.20) = Nat.4
} by {
    digit_zero_lt_ten
    Nat.0 < Nat.10
    digit_two_lt_ten
    Nat.2 < Nat.10
    happy_sum_base_mul_add_digit(Nat.10, Nat.2, Nat.0)
    happy_sum(Nat.10, Nat.10 * Nat.2 + Nat.0) =
        happy_sum(Nat.10, Nat.2) + Nat.0 * Nat.0
    happy_sum(Nat.10, Nat.20) = happy_sum(Nat.10, Nat.2) + Nat.0 * Nat.0
    happy_sum_2
    happy_sum(Nat.10, Nat.2) = Nat.4
    Nat.0 * Nat.0 = Nat.0
    happy_sum(Nat.10, Nat.20) = Nat.4 + Nat.0
    add_zero_right(Nat.4)
    Nat.4 + Nat.0 = Nat.4
    happy_sum(Nat.10, Nat.20) = Nat.4
}

/// The sum of the squares of the digits of `37` is `58`.
theorem happy_sum_37 {
    happy_sum(Nat.10, Nat.37) = Nat.58
} by {
    digit_seven_lt_ten
    Nat.7 < Nat.10
    digit_three_lt_ten
    Nat.3 < Nat.10
    happy_sum_base_mul_add_digit(Nat.10, Nat.3, Nat.7)
    happy_sum(Nat.10, Nat.10 * Nat.3 + Nat.7) =
        happy_sum(Nat.10, Nat.3) + Nat.7 * Nat.7
    happy_sum(Nat.10, Nat.37) = happy_sum(Nat.10, Nat.3) + Nat.7 * Nat.7
    happy_sum_3
    happy_sum(Nat.10, Nat.3) = Nat.9
    nat_mul_7_7
    Nat.7 * Nat.7 = Nat.49
    happy_sum(Nat.10, Nat.37) = Nat.9 + Nat.49
    nat_add_9_49
    happy_sum(Nat.10, Nat.37) = Nat.58
}

/// The sum of the squares of the digits of `42` is `20`.
theorem happy_sum_42 {
    happy_sum(Nat.10, Nat.42) = Nat.20
} by {
    digit_two_lt_ten
    Nat.2 < Nat.10
    digit_four_lt_ten
    Nat.4 < Nat.10
    happy_sum_base_mul_add_digit(Nat.10, Nat.4, Nat.2)
    happy_sum(Nat.10, Nat.10 * Nat.4 + Nat.2) =
        happy_sum(Nat.10, Nat.4) + Nat.2 * Nat.2
    happy_sum(Nat.10, Nat.42) = happy_sum(Nat.10, Nat.4) + Nat.2 * Nat.2
    happy_sum_4
    happy_sum(Nat.10, Nat.4) = Nat.16
    nat_mul_2_2
    Nat.2 * Nat.2 = Nat.4
    happy_sum(Nat.10, Nat.42) = Nat.16 + Nat.4
    nat_add_16_4
    happy_sum(Nat.10, Nat.42) = Nat.20
}

/// The sum of the squares of the digits of `49` is `97`.
theorem happy_sum_49 {
    happy_sum(Nat.10, Nat.49) = Nat.97
} by {
    digit_nine_lt_ten
    Nat.9 < Nat.10
    digit_one_lt_ten
    Nat.1 < Nat.10
    happy_sum_base_mul_add_digit(Nat.10, Nat.4, Nat.9)
    happy_sum(Nat.10, Nat.10 * Nat.4 + Nat.9) =
        happy_sum(Nat.10, Nat.4) + Nat.9 * Nat.9
    happy_sum(Nat.10, Nat.49) = happy_sum(Nat.10, Nat.4) + Nat.9 * Nat.9
    happy_sum_4
    happy_sum(Nat.10, Nat.4) = Nat.16
    nat_mul_9_9
    Nat.9 * Nat.9 = Nat.81
    happy_sum(Nat.10, Nat.49) = Nat.16 + Nat.81
    Nat.16 + Nat.81 = Nat.97
    happy_sum(Nat.10, Nat.49) = Nat.97
}

/// The sum of the squares of the digits of `58` is `89`.
theorem happy_sum_58 {
    happy_sum(Nat.10, Nat.58) = Nat.89
} by {
    digit_eight_lt_ten
    Nat.8 < Nat.10
    digit_five_lt_ten
    Nat.5 < Nat.10
    happy_sum_base_mul_add_digit(Nat.10, Nat.5, Nat.8)
    happy_sum(Nat.10, Nat.10 * Nat.5 + Nat.8) =
        happy_sum(Nat.10, Nat.5) + Nat.8 * Nat.8
    happy_sum(Nat.10, Nat.58) = happy_sum(Nat.10, Nat.5) + Nat.8 * Nat.8
    happy_sum_5
    happy_sum(Nat.10, Nat.5) = Nat.25
    nat_mul_8_8
    Nat.8 * Nat.8 = Nat.64
    happy_sum(Nat.10, Nat.58) = Nat.25 + Nat.64
    nat_add_25_64
    happy_sum(Nat.10, Nat.58) = Nat.89
}

/// The sum of the squares of the digits of `89` is `145`.
theorem happy_sum_89 {
    happy_sum(Nat.10, Nat.89) = Nat.145
} by {
    digit_nine_lt_ten
    Nat.9 < Nat.10
    digit_eight_lt_ten
    Nat.8 < Nat.10
    happy_sum_base_mul_add_digit(Nat.10, Nat.8, Nat.9)
    happy_sum(Nat.10, Nat.10 * Nat.8 + Nat.9) =
        happy_sum(Nat.10, Nat.8) + Nat.9 * Nat.9
    happy_sum(Nat.10, Nat.89) = happy_sum(Nat.10, Nat.8) + Nat.9 * Nat.9
    happy_sum_8
    happy_sum(Nat.10, Nat.8) = Nat.64
    nat_mul_9_9
    Nat.9 * Nat.9 = Nat.81
    happy_sum(Nat.10, Nat.89) = Nat.64 + Nat.81
    add_comm(Nat.64, Nat.81)
    Nat.64 + Nat.81 = Nat.81 + Nat.64
    nat_add_81_64
    Nat.81 + Nat.64 = Nat.145
    happy_sum(Nat.10, Nat.89) = Nat.145
}

/// The sum of the squares of the digits of `97` is `130`.
theorem happy_sum_97 {
    happy_sum(Nat.10, Nat.97) = Nat.130
} by {
    digit_seven_lt_ten
    Nat.7 < Nat.10
    digit_nine_lt_ten
    Nat.9 < Nat.10
    happy_sum_base_mul_add_digit(Nat.10, Nat.9, Nat.7)
    happy_sum(Nat.10, Nat.10 * Nat.9 + Nat.7) =
        happy_sum(Nat.10, Nat.9) + Nat.7 * Nat.7
    happy_sum(Nat.10, Nat.97) = happy_sum(Nat.10, Nat.9) + Nat.7 * Nat.7
    happy_sum_9
    happy_sum(Nat.10, Nat.9) = Nat.81
    nat_mul_7_7
    Nat.7 * Nat.7 = Nat.49
    happy_sum(Nat.10, Nat.97) = Nat.81 + Nat.49
    nat_add_81_49
    happy_sum(Nat.10, Nat.97) = Nat.130
}

/// The sum of the squares of the digits of `130` is `10`.
theorem happy_sum_130 {
    happy_sum(Nat.10, Nat.130) = Nat.10
} by {
    digit_zero_lt_ten
    Nat.0 < Nat.10
    digit_one_lt_ten
    Nat.1 < Nat.10
    happy_sum_base_mul_add_digit(Nat.10, Nat.13, Nat.0)
    happy_sum(Nat.10, Nat.10 * Nat.13 + Nat.0) =
        happy_sum(Nat.10, Nat.13) + Nat.0 * Nat.0
    happy_sum(Nat.10, Nat.130) = happy_sum(Nat.10, Nat.13) + Nat.0 * Nat.0
    happy_sum_13
    happy_sum(Nat.10, Nat.13) = Nat.10
    Nat.0 * Nat.0 = Nat.0
    happy_sum(Nat.10, Nat.130) = Nat.10 + Nat.0
    add_zero_right(Nat.10)
    Nat.10 + Nat.0 = Nat.10
    happy_sum(Nat.10, Nat.130) = Nat.10
}

/// The sum of the squares of the digits of `145` is `42`.
theorem happy_sum_145 {
    happy_sum(Nat.10, Nat.145) = Nat.42
} by {
    digit_five_lt_ten
    Nat.5 < Nat.10
    digit_one_lt_ten
    Nat.1 < Nat.10
    happy_sum_base_mul_add_digit(Nat.10, Nat.14, Nat.5)
    happy_sum(Nat.10, Nat.10 * Nat.14 + Nat.5) =
        happy_sum(Nat.10, Nat.14) + Nat.5 * Nat.5
    happy_sum(Nat.10, Nat.145) = happy_sum(Nat.10, Nat.14) + Nat.5 * Nat.5
    happy_sum_14
    happy_sum(Nat.10, Nat.14) = Nat.17
    nat_mul_5_5
    Nat.5 * Nat.5 = Nat.25
    happy_sum(Nat.10, Nat.145) = Nat.17 + Nat.25
    nat_add_17_25
    happy_sum(Nat.10, Nat.145) = Nat.42
}

/// Iterating the happy function `k` times from `n`.
define happy_iter(p: Nat, n: Nat, k: Nat) -> Nat {
    match k {
        Nat.zero {
            n
        }
        Nat.suc(m) {
            happy_iter(p, happy_sum(p, n), m)
        }
    }
}

/// True if iterating the happy function from `n` eventually reaches `1`.
define is_happy(p: Nat, n: Nat) -> Bool {
    exists(k: Nat) {
        happy_iter(p, n, k) = Nat.1
    }
}

/// One step of the happy iteration from `1` stays at `1`.
theorem happy_iter_1 {
    happy_iter(Nat.10, Nat.1, Nat.0) = Nat.1
}

/// Iterating the happy function from `7` five times reaches `1`:
/// `7, 49, 97, 130, 10, 1`.
theorem happy_iter_7_reaches_1 {
    happy_iter(Nat.10, Nat.7, Nat.5) = Nat.1
} by {
    happy_iter(Nat.10, Nat.7, Nat.5) =
        happy_iter(Nat.10, happy_sum(Nat.10, Nat.7), Nat.4)
    happy_sum_7
    happy_sum(Nat.10, Nat.7) = Nat.49
    happy_iter(Nat.10, Nat.7, Nat.5) = happy_iter(Nat.10, Nat.49, Nat.4)
    happy_iter(Nat.10, Nat.49, Nat.4) =
        happy_iter(Nat.10, happy_sum(Nat.10, Nat.49), Nat.3)
    happy_sum_49
    happy_sum(Nat.10, Nat.49) = Nat.97
    happy_iter(Nat.10, Nat.7, Nat.5) = happy_iter(Nat.10, Nat.97, Nat.3)
    happy_iter(Nat.10, Nat.97, Nat.3) =
        happy_iter(Nat.10, happy_sum(Nat.10, Nat.97), Nat.2)
    happy_sum_97
    happy_sum(Nat.10, Nat.97) = Nat.130
    happy_iter(Nat.10, Nat.7, Nat.5) = happy_iter(Nat.10, Nat.130, Nat.2)
    happy_iter(Nat.10, Nat.130, Nat.2) =
        happy_iter(Nat.10, happy_sum(Nat.10, Nat.130), Nat.1)
    happy_sum_130
    happy_sum(Nat.10, Nat.130) = Nat.10
    happy_iter(Nat.10, Nat.7, Nat.5) = happy_iter(Nat.10, Nat.10, Nat.1)
    happy_iter(Nat.10, Nat.10, Nat.1) =
        happy_iter(Nat.10, happy_sum(Nat.10, Nat.10), Nat.0)
    happy_sum_10
    happy_sum(Nat.10, Nat.10) = Nat.1
    happy_iter(Nat.10, Nat.7, Nat.5) = happy_iter(Nat.10, Nat.1, Nat.0)
    happy_iter(Nat.10, Nat.1, Nat.0) = Nat.1
    happy_iter(Nat.10, Nat.7, Nat.5) = Nat.1
}

/// Iterating the happy function from `4` eight times returns to `4`: the orbit
/// `4, 16, 37, 58, 89, 145, 42, 20, 4` is a cycle.
theorem happy_iter_4_cycle {
    happy_iter(Nat.10, Nat.4, Nat.8) = Nat.4
} by {
    happy_iter(Nat.10, Nat.4, Nat.8) =
        happy_iter(Nat.10, happy_sum(Nat.10, Nat.4), Nat.7)
    happy_sum_4
    happy_sum(Nat.10, Nat.4) = Nat.16
    happy_iter(Nat.10, Nat.4, Nat.8) = happy_iter(Nat.10, Nat.16, Nat.7)
    happy_iter(Nat.10, Nat.16, Nat.7) =
        happy_iter(Nat.10, happy_sum(Nat.10, Nat.16), Nat.6)
    happy_sum_16
    happy_sum(Nat.10, Nat.16) = Nat.37
    happy_iter(Nat.10, Nat.4, Nat.8) = happy_iter(Nat.10, Nat.37, Nat.6)
    happy_iter(Nat.10, Nat.37, Nat.6) =
        happy_iter(Nat.10, happy_sum(Nat.10, Nat.37), Nat.5)
    happy_sum_37
    happy_sum(Nat.10, Nat.37) = Nat.58
    happy_iter(Nat.10, Nat.4, Nat.8) = happy_iter(Nat.10, Nat.58, Nat.5)
    happy_iter(Nat.10, Nat.58, Nat.5) =
        happy_iter(Nat.10, happy_sum(Nat.10, Nat.58), Nat.4)
    happy_sum_58
    happy_sum(Nat.10, Nat.58) = Nat.89
    happy_iter(Nat.10, Nat.4, Nat.8) = happy_iter(Nat.10, Nat.89, Nat.4)
    happy_iter(Nat.10, Nat.89, Nat.4) =
        happy_iter(Nat.10, happy_sum(Nat.10, Nat.89), Nat.3)
    happy_sum_89
    happy_sum(Nat.10, Nat.89) = Nat.145
    happy_iter(Nat.10, Nat.4, Nat.8) = happy_iter(Nat.10, Nat.145, Nat.3)
    happy_iter(Nat.10, Nat.145, Nat.3) =
        happy_iter(Nat.10, happy_sum(Nat.10, Nat.145), Nat.2)
    happy_sum_145
    happy_sum(Nat.10, Nat.145) = Nat.42
    happy_iter(Nat.10, Nat.4, Nat.8) = happy_iter(Nat.10, Nat.42, Nat.2)
    happy_iter(Nat.10, Nat.42, Nat.2) =
        happy_iter(Nat.10, happy_sum(Nat.10, Nat.42), Nat.1)
    happy_sum_42
    happy_sum(Nat.10, Nat.42) = Nat.20
    happy_iter(Nat.10, Nat.4, Nat.8) = happy_iter(Nat.10, Nat.20, Nat.1)
    happy_iter(Nat.10, Nat.20, Nat.1) =
        happy_iter(Nat.10, happy_sum(Nat.10, Nat.20), Nat.0)
    happy_sum_20
    happy_sum(Nat.10, Nat.20) = Nat.4
    happy_iter(Nat.10, Nat.4, Nat.8) = happy_iter(Nat.10, Nat.4, Nat.0)
    happy_iter(Nat.10, Nat.4, Nat.0) = Nat.4
    happy_iter(Nat.10, Nat.4, Nat.8) = Nat.4
}

/// If the happy function fixes `1`, one step of the iteration from `1` stays
/// at `1`.
theorem happy_iter_step_one_fix(p: Nat, j: Nat) {
    happy_sum(p, Nat.1) = Nat.1 implies
        happy_iter(p, Nat.1, j.suc) = happy_iter(p, Nat.1, j)
} by {
    if happy_sum(p, Nat.1) = Nat.1 {
        happy_iter(p, Nat.1, j.suc) =
            happy_iter(p, happy_sum(p, Nat.1), j)
        happy_iter(p, Nat.1, j.suc) = happy_iter(p, Nat.1, j)
    }
}

/// The iteration from `1` stays at `1` forever, whenever the happy function
/// fixes `1`.
theorem happy_iter_one_stays(p: Nat, k: Nat) {
    happy_sum(p, Nat.1) = Nat.1 implies happy_iter(p, Nat.1, k) = Nat.1
} by {
    if happy_sum(p, Nat.1) = Nat.1 {
        define pred(j: Nat) -> Bool {
            happy_iter(p, Nat.1, j) = Nat.1
        }
        happy_iter(p, Nat.1, Nat.0) = Nat.1
        pred(Nat.0)
        forall(j: Nat) {
            if pred(j) {
                happy_iter_step_one_fix(p, j)
                happy_iter(p, Nat.1, j.suc) = happy_iter(p, Nat.1, j)
                pred(j) = (happy_iter(p, Nat.1, j) = Nat.1)
                let h: Bool = happy_iter(p, Nat.1, j) = Nat.1
                h
                happy_iter(p, Nat.1, j) = Nat.1
                happy_iter(p, Nat.1, j.suc) = Nat.1
                pred(j.suc)
            }
        }
        pred(k)
        pred(k) = (happy_iter(p, Nat.1, k) = Nat.1)
        let h: Bool = happy_iter(p, Nat.1, k) = Nat.1
        h
        happy_iter(p, Nat.1, k) = Nat.1
    }
}

/// In base ten the iteration from `1` stays at `1` forever.
theorem happy_iter_ten_one_stays(k: Nat) {
    happy_iter(Nat.10, Nat.1, k) = Nat.1
} by {
    happy_sum_1
    happy_sum(Nat.10, Nat.1) = Nat.1
    happy_iter_one_stays(Nat.10, k)
    happy_iter(Nat.10, Nat.1, k) = Nat.1
}

/// One is happy.
theorem is_happy_1 {
    is_happy(Nat.10, Nat.1)
} by {
    happy_iter(Nat.10, Nat.1, Nat.0) = Nat.1
    exists(k: Nat) {
        happy_iter(Nat.10, Nat.1, k) = Nat.1
    }
    is_happy(Nat.10, Nat.1) = exists(k: Nat) {
        happy_iter(Nat.10, Nat.1, k) = Nat.1
    }
    is_happy(Nat.10, Nat.1)
}

/// Seven is happy.
theorem is_happy_7 {
    is_happy(Nat.10, Nat.7)
} by {
    happy_iter_7_reaches_1
    happy_iter(Nat.10, Nat.7, Nat.5) = Nat.1
    exists(k: Nat) {
        happy_iter(Nat.10, Nat.7, k) = Nat.1
    }
    is_happy(Nat.10, Nat.7) = exists(k: Nat) {
        happy_iter(Nat.10, Nat.7, k) = Nat.1
    }
    is_happy(Nat.10, Nat.7)
}

/// The sum of the squares of the digits of a number below ten is at most `81`.
theorem happy_sum_single_digit_at_most_81(n: Nat) {
    n < Nat.10 implies happy_sum(Nat.10, n) <= Nat.81
} by {
    if n < Nat.10 {
        digit_one_lt_ten
        Nat.1 < Nat.10
        happy_sum_small(Nat.10, n)
        happy_sum(Nat.10, n) = n * n
        lt_imp_lte_suc(n, Nat.10)
        n.suc <= Nat.10
        lte_cancel_suc(n, Nat.9)
        n <= Nat.9
        lte_mul_both(n, n, Nat.9)
        n * n <= n * Nat.9
        mul_comm(n, Nat.9)
        n * Nat.9 = Nat.9 * n
        n * n <= Nat.9 * n
        lte_mul_both(Nat.9, n, Nat.9)
        Nat.9 * n <= Nat.9 * Nat.9
        lte_trans(n * n, Nat.9 * n, Nat.9 * Nat.9)
        n * n <= Nat.9 * Nat.9
        nat_mul_9_9
        Nat.9 * Nat.9 = Nat.81
        n * n <= Nat.81
        happy_sum(Nat.10, n) <= Nat.81
    }
}

// /// The sum of the squares of the digits of a number with at most `k` decimal
// /// digits is at most `81 * k`. Not finished: needs an induction over `k` with
// /// the quotient bound `n.div(10) < 10.pow(k)`.
// theorem happy_sum_at_most_81k(k: Nat, n: Nat) {
//     n < Nat.10.pow(k) implies happy_sum(Nat.10, n) <= Nat.81 * k
// }

// /// Four is not happy: its orbit enters the cycle
// /// `4, 16, 37, 58, 89, 145, 42, 20` and never reaches `1`. Not finished:
// /// proving that the cycle is never left and that `1` is not in it needs an
// /// induction over the orbit together with a disjointness argument.
// theorem not_is_happy_4 {
//     not is_happy(Nat.10, Nat.4)
// }
