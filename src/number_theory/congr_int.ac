from number_theory.congruence import Nat, mod_add_mul
from nat import add_mod, mod_by_zero
from int import Int, abs, abs_from_nat, add_from_nat, mul_nat_from_nat_left,
    mul_nat_from_nat_right, div_imp_div_abs
from zmod import int_mod_rel
numerals Int

/// Bridge helper: the embedding of a product factors as a product of embeddings.
theorem mul_from_nat(a: Nat, b: Nat) {
    Int.from_nat(a) * Int.from_nat(b) = Int.from_nat(a * b)
} by {
    mul_nat_from_nat_right(Int.from_nat(a), b)
    mul_nat_from_nat_left(a, b)
}

/// Helper: when the right operand is no larger, the integer subtraction of
/// embedded naturals coincides with the embedded truncating Nat subtraction.
theorem int_from_nat_sub_eq(a: Nat, b: Nat) {
    b <= a implies Int.from_nat(a) - Int.from_nat(b) = Int.from_nat(a - b)
} by {
    if b <= a {
        let d: Nat satisfy { b + d = a }
        Int.from_nat(d) = Int.from_nat(a - b)
        add_from_nat(b, d)
        Int.from_nat(d) + Int.from_nat(b) - Int.from_nat(b) = Int.from_nat(a) - Int.from_nat(b)
        Int.from_nat(a) - Int.from_nat(b) = Int.from_nat(a - b)
    }
}

/// Helper: if n divides a difference of naturals (with the larger first), the
/// smaller and larger leave the same remainder modulo n.
theorem nat_divides_diff_imp_congr(a: Nat, b: Nat, n: Nat) {
    b <= a and n.divides(a - b) implies a.congr_mod(b, n)
} by {
    if b <= a and n.divides(a - b) {
        let k: Nat satisfy { n * k = a - b }
        n * k = k * n
        k * n = a - b
        k * n + b = a
        mod_add_mul(k, n, b)
        a.mod(n) = b.mod(n)
    }
}

/// Sub-lemma for the backward bridge when b <= a.
theorem int_mod_rel_imp_nat_congr_mod_le(a: Nat, b: Nat, n: Nat) {
    b <= a and int_mod_rel(n, Int.from_nat(a), Int.from_nat(b)) implies a.congr_mod(b, n)
} by {
    if b <= a and int_mod_rel(n, Int.from_nat(a), Int.from_nat(b)) {
        int_from_nat_sub_eq(a, b)
        div_imp_div_abs(Int.from_nat(n), Int.from_nat(a - b))
        abs(Int.from_nat(n)).divides(abs(Int.from_nat(a - b)))
        abs(Int.from_nat(n)) = n
        abs(Int.from_nat(a - b)) = a - b
        nat_divides_diff_imp_congr(a, b, n)
        a.congr_mod(b, n)
    }
}

/// Backward direction of the bridge: the Int divisibility form implies Nat congruence.
theorem int_mod_rel_imp_nat_congr_mod(a: Nat, b: Nat, n: Nat) {
    int_mod_rel(n, Int.from_nat(a), Int.from_nat(b)) implies a.congr_mod(b, n)
} by {
    if int_mod_rel(n, Int.from_nat(a), Int.from_nat(b)) {
        let ai: Int = Int.from_nat(a)
        let bi: Int = Int.from_nat(b)
        if b <= a {
            int_mod_rel_imp_nat_congr_mod_le(a, b, n)
        } else {
            a < b
            a <= b
            let d: Int satisfy { d * Int.from_nat(n) = ai - bi }
            -d * Int.from_nat(n) = bi - ai
            Int.from_nat(n).divides(bi - ai)
            int_mod_rel(n, bi, ai)
            int_mod_rel_imp_nat_congr_mod_le(b, a, n)
            b.congr_mod(a, n)
            a.mod(n) = b.mod(n)
        }
    }
}

/// Forward direction of the bridge: Nat congruence implies the Int divisibility form.
theorem nat_congr_mod_imp_int_mod_rel(a: Nat, b: Nat, n: Nat) {
    a.congr_mod(b, n) implies int_mod_rel(n, Int.from_nat(a), Int.from_nat(b))
} by {
    if a.congr_mod(b, n) {
        a.mod(n) = b.mod(n)
        if n = Nat.0 {
            mod_by_zero(a)
            mod_by_zero(b)
            Int.from_nat(a) = Int.from_nat(b)
            int_mod_rel(n, Int.from_nat(a), Int.from_nat(b))
        } else {
            let r: Nat = a.mod(n)
            add_mod(a, n)
            let qa: Nat satisfy { qa * n + a.mod(n) = a }
            add_mod(b, n)
            let qb: Nat satisfy { qb * n + b.mod(n) = b }
            let ai: Int = Int.from_nat(a)
            let bi: Int = Int.from_nat(b)
            let ni: Int = Int.from_nat(n)
            let ri: Int = Int.from_nat(r)
            let qai: Int = Int.from_nat(qa)
            let qbi: Int = Int.from_nat(qb)
            mul_from_nat(qa, n)
            mul_from_nat(qb, n)
            add_from_nat(qa * n, r)
            add_from_nat(qb * n, r)
            (qai * ni + ri) - (qbi * ni + ri) = qai * ni - qbi * ni
            (qai - qbi) * ni = ni * (qai - qbi)
            ni * (qai - qbi) = ai - bi
            ni.divides(ai - bi)
            int_mod_rel(n, Int.from_nat(a), Int.from_nat(b))
        }
    }
}

/// Bridge between Nat and Int congruence, in iff form.
theorem nat_congr_mod_iff_int_mod_rel(a: Nat, b: Nat, n: Nat) {
    a.congr_mod(b, n) = int_mod_rel(n, Int.from_nat(a), Int.from_nat(b))
} by {
    if a.congr_mod(b, n) {
        nat_congr_mod_imp_int_mod_rel(a, b, n)
    }
    if int_mod_rel(n, Int.from_nat(a), Int.from_nat(b)) {
        int_mod_rel_imp_nat_congr_mod(a, b, n)
    }
}
