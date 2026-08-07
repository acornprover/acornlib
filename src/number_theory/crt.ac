from number_theory.coprime import coprime_one_left, coprime_mul, coprime_comm
from number_theory.bezout import nat_bezout
from nat import divides_self, divides_mul, mul_to_zero
from number_theory.congr_int import nat_congr_mod_iff_int_mod_rel
from number_theory.congruence import Nat, congr_mod_trans
from nat import gcd_mul_lcm
from number_theory.lcm import lcm_divides_of_common
from int import Int, abs, div_from_nat, div_trans, div_imp_div_abs, div_abs_imp_div
from data.int.int_residue import int_has_nat_residue
from zmod import int_mod_rel, int_mod_rel_is_equivalence
from data.basic.relation_basic import is_transitive
numerals Int

/// Half of CRT: the chosen witness c = a + (b-a)*x*m is congruent to a modulo m.
theorem crt_witness_mod_m(m: Nat, a: Int, b: Int, x: Int) {
    Int.from_nat(m).divides(a + (b - a) * x * Int.from_nat(m) - a)
} by {
    let mi: Int = Int.from_nat(m)
    let c: Int = a + (b - a) * x * mi
    c - a = (b - a) * x * mi
    (b - a) * x * mi = mi * ((b - a) * x)
    mi.divides(c - a)
}

/// Half of CRT: under the Bezout identity x*m + y*n = 1, the witness
/// c = a + (b-a)*x*m is congruent to b modulo n.
theorem crt_witness_mod_n(m: Nat, n: Nat, a: Int, b: Int, x: Int, y: Int) {
    x * Int.from_nat(m) + y * Int.from_nat(n) = Int.1
        implies Int.from_nat(n).divides(a + (b - a) * x * Int.from_nat(m) - b)
} by {
    if x * Int.from_nat(m) + y * Int.from_nat(n) = Int.1 {
        let mi: Int = Int.from_nat(m)
        let ni: Int = Int.from_nat(n)
        let bma: Int = b - a
        bma + a = b
        let c: Int = a + bma * x * mi
        c - b = a + bma * x * mi - (bma + a)
        a + bma * x * mi - (bma + a) = bma * x * mi - bma
        c - b = bma * x * mi - bma
        bma * x * mi - bma = bma * (x * mi - Int.1)
        c - b = bma * (x * mi - Int.1)
        x * mi - Int.1 = -(y * ni)
        bma * (x * mi - Int.1) = bma * -(y * ni)
        bma * -(y * ni) = -(bma * y * ni)
        c - b = -(bma * y * ni)
        -(bma * y * ni) = ni * (-(bma * y))
        ni.divides(c - b)
        Int.from_nat(n).divides(a + (b - a) * x * Int.from_nat(m) - b)
    }
}

/// Two-modulus Chinese remainder theorem in Int form: when m and n are
/// coprime on Nat, every pair of integer residue requirements has a
/// simultaneous integer solution.
theorem int_crt_two_moduli(m: Nat, n: Nat, a: Int, b: Int) {
    m.coprime(n) implies exists(c: Int) {
        int_mod_rel(m, c, a) and int_mod_rel(n, c, b)
    }
} by {
    if m.coprime(n) {
        m.gcd(n) = Nat.1
        nat_bezout(m, n)
        let (x: Int, y: Int) satisfy {
            x * Int.from_nat(m) + y * Int.from_nat(n) = Int.from_nat(m.gcd(n))
        }
        Int.from_nat(m.gcd(n)) = Int.from_nat(Nat.1)
        Int.from_nat(Nat.1) = Int.1
        x * Int.from_nat(m) + y * Int.from_nat(n) = Int.1
        let c: Int = a + (b - a) * x * Int.from_nat(m)
        crt_witness_mod_m(m, a, b, x)
        crt_witness_mod_n(m, n, a, b, x, y)
        Int.from_nat(m).divides(c - a)
        Int.from_nat(n).divides(c - b)
        int_mod_rel(m, c, a)
        int_mod_rel(n, c, b)
        exists(c0: Int) { int_mod_rel(m, c0, a) and int_mod_rel(n, c0, b) }
    }
}

/// When two coprime moduli both divide a Nat, so does their product.
theorem nat_coprime_combine(m: Nat, n: Nat, k: Nat) {
    m.coprime(n) and m.divides(k) and n.divides(k) implies (m * n).divides(k)
} by {
    if m.coprime(n) and m.divides(k) and n.divides(k) {
        lcm_divides_of_common(m, n, k)
        m.lcm(n).divides(k)
        m.gcd(n) = Nat.1
        gcd_mul_lcm(m, n)
        Nat.1 * m.lcm(n) = m * n
        m.lcm(n) = m * n
        (m * n).divides(k)
    }
}

/// On the integers: when two coprime moduli both witness a congruence, so
/// does their product.
theorem int_mod_rel_combine_coprime(m: Nat, n: Nat, x: Int, y: Int) {
    m.coprime(n) and int_mod_rel(m, x, y) and int_mod_rel(n, x, y)
        implies int_mod_rel(m * n, x, y)
} by {
    if m.coprime(n) and int_mod_rel(m, x, y) and int_mod_rel(n, x, y) {
        Int.from_nat(m).divides(x - y)
        Int.from_nat(n).divides(x - y)
        div_imp_div_abs(Int.from_nat(m), x - y)
        div_imp_div_abs(Int.from_nat(n), x - y)
        abs(Int.from_nat(m)) = m
        abs(Int.from_nat(n)) = n
        m.divides(abs(x - y))
        n.divides(abs(x - y))
        nat_coprime_combine(m, n, abs(x - y))
        (m * n).divides(abs(x - y))
        abs(Int.from_nat(m * n)) = m * n
        abs(Int.from_nat(m * n)).divides(abs(x - y))
        div_abs_imp_div(Int.from_nat(m * n), x - y)
        Int.from_nat(m * n).divides(x - y)
    }
}

/// CRT uniqueness on the integers: any two integer simultaneous solutions of
/// a coprime two-modulus congruence system agree modulo the product.
theorem int_crt_unique(m: Nat, n: Nat, c1: Int, c2: Int, a: Int, b: Int) {
    m.coprime(n)
        and int_mod_rel(m, c1, a) and int_mod_rel(n, c1, b)
        and int_mod_rel(m, c2, a) and int_mod_rel(n, c2, b)
        implies int_mod_rel(m * n, c1, c2)
} by {
    if m.coprime(n)
        and int_mod_rel(m, c1, a) and int_mod_rel(n, c1, b)
        and int_mod_rel(m, c2, a) and int_mod_rel(n, c2, b) {
        int_mod_rel_is_equivalence(m)
        is_transitive(int_mod_rel(m))
        int_mod_rel_is_equivalence(n)
        is_transitive(int_mod_rel(n))
        int_mod_rel(m, a, c2)
        int_mod_rel(m, c1, c2)
        int_mod_rel(n, b, c2)
        int_mod_rel(n, c1, c2)
        int_mod_rel_combine_coprime(m, n, c1, c2)
        int_mod_rel(m * n, c1, c2)
    }
}

/// Descending the modulus along divisibility: a congruence modulo a multiple
/// of m is also a congruence modulo m.
theorem int_mod_rel_descend(m: Nat, k: Nat, x: Int, y: Int) {
    m.divides(k) and int_mod_rel(k, x, y) implies int_mod_rel(m, x, y)
} by {
    if m.divides(k) and int_mod_rel(k, x, y) {
        div_from_nat(m, k)
        Int.from_nat(m).divides(Int.from_nat(k))
        Int.from_nat(k).divides(x - y)
        div_trans(Int.from_nat(m), Int.from_nat(k), x - y)
        Int.from_nat(m).divides(x - y)
    }
}

/// Two-modulus Chinese remainder theorem on the naturals: for coprime
/// positive moduli, every pair of natural-number residue requirements has a
/// simultaneous Nat solution.
theorem nat_crt_two_moduli(m: Nat, n: Nat, a: Nat, b: Nat) {
    m.coprime(n) and m != Nat.0 and n != Nat.0 implies
        exists(c: Nat) { c.congr_mod(a, m) and c.congr_mod(b, n) }
} by {
    if m.coprime(n) and m != Nat.0 and n != Nat.0 {
        int_crt_two_moduli(m, n, Int.from_nat(a), Int.from_nat(b))
        let ci: Int satisfy {
            int_mod_rel(m, ci, Int.from_nat(a)) and int_mod_rel(n, ci, Int.from_nat(b))
        }
        mul_to_zero(m, n)
        m * n != Nat.0
        int_has_nat_residue(ci, m * n)
        let c: Nat satisfy { int_mod_rel(m * n, ci, Int.from_nat(c)) }
        // Descend modulus from m*n to m (since m | m*n).
        divides_self(m)
        divides_mul(m, n, m)
        m.divides(m * n)
        int_mod_rel_descend(m, m * n, ci, Int.from_nat(c))
        int_mod_rel(m, ci, Int.from_nat(c))
        // Combine with int_mod_rel(m, ci, Int.from_nat(a)) by symmetry/transitivity.
        int_mod_rel_is_equivalence(m)
        is_transitive(int_mod_rel(m))
        int_mod_rel(m, Int.from_nat(c), ci)
        int_mod_rel(m, Int.from_nat(c), Int.from_nat(a))
        nat_congr_mod_iff_int_mod_rel(c, a, m)
        c.congr_mod(a, m)
        // Same for n: descend and combine.
        divides_self(n)
        divides_mul(n, m, n)
        n.divides(n * m)
        n * m = m * n
        n.divides(m * n)
        int_mod_rel_descend(n, m * n, ci, Int.from_nat(c))
        int_mod_rel(n, ci, Int.from_nat(c))
        int_mod_rel_is_equivalence(n)
        is_transitive(int_mod_rel(n))
        int_mod_rel(n, Int.from_nat(c), ci)
        int_mod_rel(n, Int.from_nat(c), Int.from_nat(b))
        nat_congr_mod_iff_int_mod_rel(c, b, n)
        c.congr_mod(b, n)
        c.congr_mod(a, m) and c.congr_mod(b, n)
        exists(c0: Nat) { c0.congr_mod(a, m) and c0.congr_mod(b, n) }
    }
}


/// Three-modulus Chinese remainder theorem on the naturals: when m, n, p
/// are positive and pairwise coprime, every triple of natural-number
/// residue requirements has a simultaneous Nat solution.
theorem nat_crt_three_moduli(m: Nat, n: Nat, p: Nat, a: Nat, b: Nat, c: Nat) {
    m.coprime(n) and m.coprime(p) and n.coprime(p)
        and m != Nat.0 and n != Nat.0 and p != Nat.0
        implies exists(x: Nat) {
            x.congr_mod(a, m) and x.congr_mod(b, n) and x.congr_mod(c, p)
        }
} by {
    if m.coprime(n) and m.coprime(p) and n.coprime(p)
        and m != Nat.0 and n != Nat.0 and p != Nat.0 {
        // Solve the first two moduli.
        nat_crt_two_moduli(m, n, a, b)
        let y: Nat satisfy { y.congr_mod(a, m) and y.congr_mod(b, n) }
        // Combine the (m, n) solution with the third modulus p.
        // Need (m * n).coprime(p).
        coprime_comm(m, p)
        coprime_comm(n, p)
        p.coprime(m)
        p.coprime(n)
        coprime_mul(p, m, n)
        p.coprime(m * n)
        coprime_comm(p, m * n)
        (m * n).coprime(p)
        // m * n is positive.
        mul_to_zero(m, n)
        m * n != Nat.0
        // Apply two-modulus CRT to (m*n, p, y, c).
        nat_crt_two_moduli(m * n, p, y, c)
        let x: Nat satisfy { x.congr_mod(y, m * n) and x.congr_mod(c, p) }
        // Bridge x ≡ y (mod m*n) into x ≡ y (mod m) and x ≡ y (mod n) via
        // int_mod_rel_descend, then chain with y's congruences.
        nat_congr_mod_iff_int_mod_rel(x, y, m * n)
        int_mod_rel(m * n, Int.from_nat(x), Int.from_nat(y))
        divides_self(m)
        divides_mul(m, n, m)
        m.divides(m * n)
        int_mod_rel_descend(m, m * n, Int.from_nat(x), Int.from_nat(y))
        int_mod_rel(m, Int.from_nat(x), Int.from_nat(y))
        nat_congr_mod_iff_int_mod_rel(x, y, m)
        x.congr_mod(y, m)
        divides_self(n)
        divides_mul(n, m, n)
        n.divides(n * m)
        n * m = m * n
        n.divides(m * n)
        int_mod_rel_descend(n, m * n, Int.from_nat(x), Int.from_nat(y))
        int_mod_rel(n, Int.from_nat(x), Int.from_nat(y))
        nat_congr_mod_iff_int_mod_rel(x, y, n)
        x.congr_mod(y, n)
        // Compose with y's congruences via transitivity.
        congr_mod_trans(x, y, a, m)
        x.congr_mod(a, m)
        congr_mod_trans(x, y, b, n)
        x.congr_mod(b, n)
        x.congr_mod(c, p)
        x.congr_mod(a, m) and x.congr_mod(b, n) and x.congr_mod(c, p)
    }
}

/// CRT combine on the naturals: when m and n are coprime and a is congruent to
/// b modulo each of them, a is congruent to b modulo their product.
theorem nat_congr_combine_coprime(m: Nat, n: Nat, a: Nat, b: Nat) {
    m.coprime(n) and a.congr_mod(b, m) and a.congr_mod(b, n)
        implies a.congr_mod(b, m * n)
} by {
    if m.coprime(n) and a.congr_mod(b, m) and a.congr_mod(b, n) {
        nat_congr_mod_iff_int_mod_rel(a, b, m)
        int_mod_rel(m, Int.from_nat(a), Int.from_nat(b))
        nat_congr_mod_iff_int_mod_rel(a, b, n)
        int_mod_rel(n, Int.from_nat(a), Int.from_nat(b))
        int_mod_rel_combine_coprime(m, n, Int.from_nat(a), Int.from_nat(b))
        int_mod_rel(m * n, Int.from_nat(a), Int.from_nat(b))
        nat_congr_mod_iff_int_mod_rel(a, b, m * n)
        a.congr_mod(b, m * n)
    }
}
