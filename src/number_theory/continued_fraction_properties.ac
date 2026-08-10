/// Continued-fraction properties.
///
/// Classical properties of the convergents p_n/q_n of a continued fraction
/// and of the continued fraction of √2, assembled from the library's
/// continued-fraction development:
///
///   1. The continued fraction of √2 is [1; 2, 2, 2, ...]: the integral part
///      is one and every later partial quotient is two, the first few partial
///      quotients are 1, 2, 2, the first few convergents are 1/1, 3/2, 7/5,
///      and the convergent sequence satisfies the recurrences
///      p_{n+1} = p_n + 2·q_n, q_{n+1} = p_n + q_n (cf_pell.ac) and
///      p_{n+2} = 2·p_{n+1} + p_n, q_{n+2} = 2·q_{n+1} + q_n (Section 1).
///
///   2. The Pell norm of every convergent of √2 is plus or minus one:
///      p_n² - 2·q_n² = ±1 (cf_pell.ac, restated cleanly in Section 2).
///
///   3. The determinant identity p_n·q_{n-1} - p_{n-1}·q_n = ±1, stated at
///      every index as p_{n+1}·q_n - p_n·q_{n+1} = ±1 (Section 3).
///
///   4. The irrationality of √2 in the Diophantine form x² = 2·y², restated
///      from diophantine.ac; the general theorem that √N is irrational unless
///      N is a perfect square is stated at the end of Section 4 (it needs the
///      prime-factorization form of "not a perfect square").
///
///   5. The best-approximation property of convergents is stated at the end
///      (Section 5): its ingredients (the approximation estimate and the
///      between-neighbors denominator bound) are in the library, but the full
///      statement is not yet proved.
from nat import Nat, lt_not_ref
from int import Int, abs, abs_neg, neg_sub
from rat import Rat
from real import Real
from algebra.ring.ring import alternating_sign
from number_theory.continued_fraction_convergents import continued_fraction_convergent_numerator,
    continued_fraction_convergent_denominator, continued_fraction_convergent_value,
    continued_fraction_adjacent_convergent_determinant,
    continued_fraction_adjacent_convergent_determinant_identity,
    continued_fraction_convergent_numerator_zero, continued_fraction_convergent_denominator_zero,
    positive_continued_fraction_sequence_tail
from number_theory.continued_fraction_approx import continued_fraction_alternating_sign_one_or_neg_one,
    continued_fraction_convergent_determinant_abs_one,
    continued_fraction_convergent_numerator_two_suc, continued_fraction_convergent_denominator_two_suc,
    continued_fraction_real_convergent_value, continued_fraction_real_limit,
    continued_fraction_real_even_convergent, continued_fraction_real_odd_convergent,
    continued_fraction_real_even_lt_limit, continued_fraction_real_limit_lt_odd,
    continued_fraction_convergents_tail_bound_for_eps
from number_theory.pell import sqrt_two_continued_fraction_coefficients, pell_norm,
    sqrt_two_convergent_numerator_one, sqrt_two_convergent_denominator_one
from number_theory.cf_pell import cf_pell_convergent_compose_step, cf_pell_convergent_norm_one_or_neg_one,
    cf_pell_convergent_norm_abs_one
from number_theory.diophantine import no_nontrivial_sq_eq_two_sq, no_nontrivial_int_sq_eq_two_sq

// ============================================================================
// Section 1: the continued fraction of √2 is [1; 2, 2, 2, ...]
// ============================================================================

/// The integral part of the continued fraction of √2 is one.
theorem continued_fraction_properties_sqrt_two_coefficient_zero {
    sqrt_two_continued_fraction_coefficients(Nat.0) = Nat.1
} by {
    sqrt_two_continued_fraction_coefficients(Nat.0) = Nat.1
}

/// Every partial quotient after the integral part of the continued fraction
/// of √2 is two: the continued fraction is [1; 2, 2, 2, ...].
theorem continued_fraction_properties_sqrt_two_coefficient_suc(n: Nat) {
    sqrt_two_continued_fraction_coefficients(n.suc) = Nat.2
} by {
    sqrt_two_continued_fraction_coefficients(n.suc) = Nat.2
}

/// The first partial quotient of √2 is two.
theorem continued_fraction_properties_sqrt_two_coefficient_one {
    sqrt_two_continued_fraction_coefficients(Nat.1) = Nat.2
} by {
    continued_fraction_properties_sqrt_two_coefficient_suc(Nat.0)
}

/// The second partial quotient of √2 is two.
theorem continued_fraction_properties_sqrt_two_coefficient_two {
    sqrt_two_continued_fraction_coefficients(Nat.2) = Nat.2
} by {
    continued_fraction_properties_sqrt_two_coefficient_suc(Nat.1)
}

/// The first three partial quotients of the continued fraction of √2 are
/// 1, 2, 2.
theorem continued_fraction_properties_sqrt_two_first_partial_quotients {
    sqrt_two_continued_fraction_coefficients(Nat.0) = Nat.1 and
        sqrt_two_continued_fraction_coefficients(Nat.1) = Nat.2 and
        sqrt_two_continued_fraction_coefficients(Nat.2) = Nat.2
} by {
    continued_fraction_properties_sqrt_two_coefficient_zero
    sqrt_two_continued_fraction_coefficients(Nat.0) = Nat.1
    continued_fraction_properties_sqrt_two_coefficient_one
    sqrt_two_continued_fraction_coefficients(Nat.1) = Nat.2
    continued_fraction_properties_sqrt_two_coefficient_two
    sqrt_two_continued_fraction_coefficients(Nat.2) = Nat.2
}

/// The partial quotients of √2 are periodic with period two: after the
/// integral part, every quotient equals the next one.
theorem continued_fraction_properties_sqrt_two_period_two(n: Nat) {
    sqrt_two_continued_fraction_coefficients(n.suc) =
        sqrt_two_continued_fraction_coefficients(n.suc.suc)
} by {
    continued_fraction_properties_sqrt_two_coefficient_suc(n)
    sqrt_two_continued_fraction_coefficients(n.suc) = Nat.2
    continued_fraction_properties_sqrt_two_coefficient_suc(n.suc)
    sqrt_two_continued_fraction_coefficients(n.suc.suc) = Nat.2
    sqrt_two_continued_fraction_coefficients(n.suc) =
        sqrt_two_continued_fraction_coefficients(n.suc.suc)
}

/// Every coefficient of the continued fraction of √2 after the integral part
/// is positive.
theorem continued_fraction_properties_sqrt_two_coefficient_suc_positive(n: Nat) {
    Nat.0 < sqrt_two_continued_fraction_coefficients(n.suc)
} by {
    continued_fraction_properties_sqrt_two_coefficient_suc(n)
    sqrt_two_continued_fraction_coefficients(n.suc) = Nat.2
    Nat.0 < Nat.2
    Nat.0 < sqrt_two_continued_fraction_coefficients(n.suc)
}

/// Every coefficient of the continued fraction of √2 after the integral part
/// is positive, so the coefficient sequence has a positive tail.
theorem continued_fraction_properties_sqrt_two_positive_tail {
    positive_continued_fraction_sequence_tail(sqrt_two_continued_fraction_coefficients)
} by {
    forall(n: Nat) {
        continued_fraction_properties_sqrt_two_coefficient_suc_positive(n)
        Nat.0 < sqrt_two_continued_fraction_coefficients(n.suc)
    }
    positive_continued_fraction_sequence_tail(sqrt_two_continued_fraction_coefficients)
}

/// The zeroth convergent of √2 is 1/1.
theorem continued_fraction_properties_sqrt_two_convergent_numerator_zero {
    continued_fraction_convergent_numerator(
        sqrt_two_continued_fraction_coefficients, Nat.0) = Nat.1
} by {
    continued_fraction_convergent_numerator_zero(sqrt_two_continued_fraction_coefficients)
    continued_fraction_convergent_numerator(
        sqrt_two_continued_fraction_coefficients, Nat.0) =
        sqrt_two_continued_fraction_coefficients(Nat.0)
    continued_fraction_properties_sqrt_two_coefficient_zero
    sqrt_two_continued_fraction_coefficients(Nat.0) = Nat.1
    continued_fraction_convergent_numerator(
        sqrt_two_continued_fraction_coefficients, Nat.0) = Nat.1
}

/// The zeroth convergent of √2 has denominator one.
theorem continued_fraction_properties_sqrt_two_convergent_denominator_zero {
    continued_fraction_convergent_denominator(
        sqrt_two_continued_fraction_coefficients, Nat.0) = Nat.1
} by {
    continued_fraction_convergent_denominator_zero(sqrt_two_continued_fraction_coefficients)
    continued_fraction_convergent_denominator(
        sqrt_two_continued_fraction_coefficients, Nat.0) = Nat.1
}

/// The second convergent of √2 is 7/5.
theorem continued_fraction_properties_sqrt_two_convergent_numerator_two {
    continued_fraction_convergent_numerator(
        sqrt_two_continued_fraction_coefficients, Nat.2) = Nat.7
} by {
    cf_pell_convergent_compose_step(Nat.1)
    continued_fraction_convergent_numerator(
            sqrt_two_continued_fraction_coefficients, Nat.1.suc) =
        continued_fraction_convergent_numerator(
                sqrt_two_continued_fraction_coefficients, Nat.1) +
            Nat.2 * continued_fraction_convergent_denominator(
                sqrt_two_continued_fraction_coefficients, Nat.1) and
    continued_fraction_convergent_denominator(
            sqrt_two_continued_fraction_coefficients, Nat.1.suc) =
        continued_fraction_convergent_numerator(
                sqrt_two_continued_fraction_coefficients, Nat.1) +
            continued_fraction_convergent_denominator(
                sqrt_two_continued_fraction_coefficients, Nat.1)
    continued_fraction_convergent_numerator(
            sqrt_two_continued_fraction_coefficients, Nat.1.suc) =
        continued_fraction_convergent_numerator(
                sqrt_two_continued_fraction_coefficients, Nat.1) +
            Nat.2 * continued_fraction_convergent_denominator(
                sqrt_two_continued_fraction_coefficients, Nat.1)
    Nat.1.suc = Nat.2
    continued_fraction_convergent_numerator(
            sqrt_two_continued_fraction_coefficients, Nat.2) =
        continued_fraction_convergent_numerator(
                sqrt_two_continued_fraction_coefficients, Nat.1) +
            Nat.2 * continued_fraction_convergent_denominator(
                sqrt_two_continued_fraction_coefficients, Nat.1)
    sqrt_two_convergent_numerator_one
    continued_fraction_convergent_numerator(
        sqrt_two_continued_fraction_coefficients, Nat.1) = Nat.3
    sqrt_two_convergent_denominator_one
    continued_fraction_convergent_denominator(
        sqrt_two_continued_fraction_coefficients, Nat.1) = Nat.2
    continued_fraction_convergent_numerator(
            sqrt_two_continued_fraction_coefficients, Nat.2) =
        Nat.3 + Nat.2 * Nat.2
    Nat.3 + Nat.2 * Nat.2 = Nat.7
    continued_fraction_convergent_numerator(
        sqrt_two_continued_fraction_coefficients, Nat.2) = Nat.7
}

/// The second convergent of √2 has denominator five.
theorem continued_fraction_properties_sqrt_two_convergent_denominator_two {
    continued_fraction_convergent_denominator(
        sqrt_two_continued_fraction_coefficients, Nat.2) = Nat.5
} by {
    cf_pell_convergent_compose_step(Nat.1)
    continued_fraction_convergent_numerator(
            sqrt_two_continued_fraction_coefficients, Nat.1.suc) =
        continued_fraction_convergent_numerator(
                sqrt_two_continued_fraction_coefficients, Nat.1) +
            Nat.2 * continued_fraction_convergent_denominator(
                sqrt_two_continued_fraction_coefficients, Nat.1) and
    continued_fraction_convergent_denominator(
            sqrt_two_continued_fraction_coefficients, Nat.1.suc) =
        continued_fraction_convergent_numerator(
                sqrt_two_continued_fraction_coefficients, Nat.1) +
            continued_fraction_convergent_denominator(
                sqrt_two_continued_fraction_coefficients, Nat.1)
    continued_fraction_convergent_denominator(
            sqrt_two_continued_fraction_coefficients, Nat.1.suc) =
        continued_fraction_convergent_numerator(
                sqrt_two_continued_fraction_coefficients, Nat.1) +
            continued_fraction_convergent_denominator(
                sqrt_two_continued_fraction_coefficients, Nat.1)
    Nat.1.suc = Nat.2
    continued_fraction_convergent_denominator(
            sqrt_two_continued_fraction_coefficients, Nat.2) =
        continued_fraction_convergent_numerator(
                sqrt_two_continued_fraction_coefficients, Nat.1) +
            continued_fraction_convergent_denominator(
                sqrt_two_continued_fraction_coefficients, Nat.1)
    sqrt_two_convergent_numerator_one
    continued_fraction_convergent_numerator(
        sqrt_two_continued_fraction_coefficients, Nat.1) = Nat.3
    sqrt_two_convergent_denominator_one
    continued_fraction_convergent_denominator(
        sqrt_two_continued_fraction_coefficients, Nat.1) = Nat.2
    continued_fraction_convergent_denominator(
            sqrt_two_continued_fraction_coefficients, Nat.2) =
        Nat.3 + Nat.2
    Nat.3 + Nat.2 = Nat.5
    continued_fraction_convergent_denominator(
        sqrt_two_continued_fraction_coefficients, Nat.2) = Nat.5
}

/// The first convergent of √2 is the rational 3/2.
theorem continued_fraction_properties_sqrt_two_convergent_value_one {
    continued_fraction_convergent_value(
        sqrt_two_continued_fraction_coefficients, Nat.1) =
        Rat.from_nat(Nat.3) / Rat.from_nat(Nat.2)
} by {
    continued_fraction_convergent_value(
            sqrt_two_continued_fraction_coefficients, Nat.1) =
        Rat.from_nat(continued_fraction_convergent_numerator(
            sqrt_two_continued_fraction_coefficients, Nat.1)) /
        Rat.from_nat(continued_fraction_convergent_denominator(
            sqrt_two_continued_fraction_coefficients, Nat.1))
    sqrt_two_convergent_numerator_one
    continued_fraction_convergent_numerator(
        sqrt_two_continued_fraction_coefficients, Nat.1) = Nat.3
    sqrt_two_convergent_denominator_one
    continued_fraction_convergent_denominator(
        sqrt_two_continued_fraction_coefficients, Nat.1) = Nat.2
    continued_fraction_convergent_value(
        sqrt_two_continued_fraction_coefficients, Nat.1) =
        Rat.from_nat(Nat.3) / Rat.from_nat(Nat.2)
}

/// The second convergent of √2 is the rational 7/5.
theorem continued_fraction_properties_sqrt_two_convergent_value_two {
    continued_fraction_convergent_value(
        sqrt_two_continued_fraction_coefficients, Nat.2) =
        Rat.from_nat(Nat.7) / Rat.from_nat(Nat.5)
} by {
    continued_fraction_convergent_value(
            sqrt_two_continued_fraction_coefficients, Nat.2) =
        Rat.from_nat(continued_fraction_convergent_numerator(
            sqrt_two_continued_fraction_coefficients, Nat.2)) /
        Rat.from_nat(continued_fraction_convergent_denominator(
            sqrt_two_continued_fraction_coefficients, Nat.2))
    continued_fraction_properties_sqrt_two_convergent_numerator_two
    continued_fraction_convergent_numerator(
        sqrt_two_continued_fraction_coefficients, Nat.2) = Nat.7
    continued_fraction_properties_sqrt_two_convergent_denominator_two
    continued_fraction_convergent_denominator(
        sqrt_two_continued_fraction_coefficients, Nat.2) = Nat.5
    continued_fraction_convergent_value(
        sqrt_two_continued_fraction_coefficients, Nat.2) =
        Rat.from_nat(Nat.7) / Rat.from_nat(Nat.5)
}

/// Each convergent of √2 is obtained from the previous one by composing with
/// the fundamental unit 1 + √2: p_{n+1} = p_n + 2·q_n and q_{n+1} = p_n + q_n.
theorem continued_fraction_properties_sqrt_two_compose_step(n: Nat) {
    continued_fraction_convergent_numerator(
            sqrt_two_continued_fraction_coefficients, n.suc) =
        continued_fraction_convergent_numerator(
            sqrt_two_continued_fraction_coefficients, n) +
            Nat.2 * continued_fraction_convergent_denominator(
                sqrt_two_continued_fraction_coefficients, n) and
    continued_fraction_convergent_denominator(
            sqrt_two_continued_fraction_coefficients, n.suc) =
        continued_fraction_convergent_numerator(
            sqrt_two_continued_fraction_coefficients, n) +
            continued_fraction_convergent_denominator(
                sqrt_two_continued_fraction_coefficients, n)
} by {
    cf_pell_convergent_compose_step(n)
    continued_fraction_convergent_numerator(
            sqrt_two_continued_fraction_coefficients, n.suc) =
        continued_fraction_convergent_numerator(
            sqrt_two_continued_fraction_coefficients, n) +
            Nat.2 * continued_fraction_convergent_denominator(
                sqrt_two_continued_fraction_coefficients, n) and
    continued_fraction_convergent_denominator(
            sqrt_two_continued_fraction_coefficients, n.suc) =
        continued_fraction_convergent_numerator(
            sqrt_two_continued_fraction_coefficients, n) +
            continued_fraction_convergent_denominator(
                sqrt_two_continued_fraction_coefficients, n)
}

/// The numerators of the convergents of √2 satisfy the recurrence
/// p_{n+2} = 2·p_{n+1} + p_n.
theorem continued_fraction_properties_sqrt_two_numerator_two_suc(n: Nat) {
    continued_fraction_convergent_numerator(
            sqrt_two_continued_fraction_coefficients, n.suc.suc) =
        Nat.2 * continued_fraction_convergent_numerator(
            sqrt_two_continued_fraction_coefficients, n.suc) +
        continued_fraction_convergent_numerator(
            sqrt_two_continued_fraction_coefficients, n)
} by {
    continued_fraction_convergent_numerator_two_suc(
        sqrt_two_continued_fraction_coefficients, n)
    continued_fraction_convergent_numerator(
            sqrt_two_continued_fraction_coefficients, n.suc.suc) =
        continued_fraction_convergent_numerator(
                sqrt_two_continued_fraction_coefficients, n.suc) *
            sqrt_two_continued_fraction_coefficients(n.suc.suc) +
        continued_fraction_convergent_numerator(
            sqrt_two_continued_fraction_coefficients, n)
    continued_fraction_properties_sqrt_two_coefficient_suc(n.suc)
    sqrt_two_continued_fraction_coefficients(n.suc.suc) = Nat.2
    continued_fraction_convergent_numerator(
            sqrt_two_continued_fraction_coefficients, n.suc.suc) =
        continued_fraction_convergent_numerator(
                sqrt_two_continued_fraction_coefficients, n.suc) *
            Nat.2 +
        continued_fraction_convergent_numerator(
            sqrt_two_continued_fraction_coefficients, n)
    continued_fraction_convergent_numerator(
            sqrt_two_continued_fraction_coefficients, n.suc) *
        Nat.2 =
        Nat.2 * continued_fraction_convergent_numerator(
            sqrt_two_continued_fraction_coefficients, n.suc)
    continued_fraction_convergent_numerator(
            sqrt_two_continued_fraction_coefficients, n.suc.suc) =
        Nat.2 * continued_fraction_convergent_numerator(
            sqrt_two_continued_fraction_coefficients, n.suc) +
        continued_fraction_convergent_numerator(
            sqrt_two_continued_fraction_coefficients, n)
}

/// The denominators of the convergents of √2 satisfy the recurrence
/// q_{n+2} = 2·q_{n+1} + q_n.
theorem continued_fraction_properties_sqrt_two_denominator_two_suc(n: Nat) {
    continued_fraction_convergent_denominator(
            sqrt_two_continued_fraction_coefficients, n.suc.suc) =
        Nat.2 * continued_fraction_convergent_denominator(
            sqrt_two_continued_fraction_coefficients, n.suc) +
        continued_fraction_convergent_denominator(
            sqrt_two_continued_fraction_coefficients, n)
} by {
    continued_fraction_convergent_denominator_two_suc(
        sqrt_two_continued_fraction_coefficients, n)
    continued_fraction_convergent_denominator(
            sqrt_two_continued_fraction_coefficients, n.suc.suc) =
        continued_fraction_convergent_denominator(
                sqrt_two_continued_fraction_coefficients, n.suc) *
            sqrt_two_continued_fraction_coefficients(n.suc.suc) +
        continued_fraction_convergent_denominator(
            sqrt_two_continued_fraction_coefficients, n)
    continued_fraction_properties_sqrt_two_coefficient_suc(n.suc)
    sqrt_two_continued_fraction_coefficients(n.suc.suc) = Nat.2
    continued_fraction_convergent_denominator(
            sqrt_two_continued_fraction_coefficients, n.suc.suc) =
        continued_fraction_convergent_denominator(
                sqrt_two_continued_fraction_coefficients, n.suc) *
            Nat.2 +
        continued_fraction_convergent_denominator(
            sqrt_two_continued_fraction_coefficients, n)
    continued_fraction_convergent_denominator(
            sqrt_two_continued_fraction_coefficients, n.suc) *
        Nat.2 =
        Nat.2 * continued_fraction_convergent_denominator(
            sqrt_two_continued_fraction_coefficients, n.suc)
    continued_fraction_convergent_denominator(
            sqrt_two_continued_fraction_coefficients, n.suc.suc) =
        Nat.2 * continued_fraction_convergent_denominator(
            sqrt_two_continued_fraction_coefficients, n.suc) +
        continued_fraction_convergent_denominator(
            sqrt_two_continued_fraction_coefficients, n)
}

/// The convergents of the continued fraction of √2 converge to its real
/// limit: for every positive epsilon, all sufficiently late convergents lie
/// within epsilon of the limit.
theorem continued_fraction_properties_sqrt_two_convergents_tail_bound(eps: Real) {
    eps.is_positive implies exists(n: Nat) {
        forall(m: Nat) {
            n <= m implies (continued_fraction_real_convergent_value(
                    sqrt_two_continued_fraction_coefficients, m) -
                continued_fraction_real_limit(
                    sqrt_two_continued_fraction_coefficients)).abs < eps
        }
    }
} by {
    if eps.is_positive {
        continued_fraction_properties_sqrt_two_positive_tail
        positive_continued_fraction_sequence_tail(
            sqrt_two_continued_fraction_coefficients)
        positive_continued_fraction_sequence_tail(
                sqrt_two_continued_fraction_coefficients) and
            eps.is_positive
        continued_fraction_convergents_tail_bound_for_eps(
            sqrt_two_continued_fraction_coefficients, eps)
        exists(n: Nat) {
            forall(m: Nat) {
                n <= m implies (continued_fraction_real_convergent_value(
                        sqrt_two_continued_fraction_coefficients, m) -
                    continued_fraction_real_limit(
                        sqrt_two_continued_fraction_coefficients)).abs < eps
            }
        }
    }
}

/// The real limit of the continued fraction of √2 lies strictly between the
/// zeroth convergent 1 and the first convergent 3/2.
theorem continued_fraction_properties_sqrt_two_limit_between_first_convergents {
    continued_fraction_real_convergent_value(
            sqrt_two_continued_fraction_coefficients, Nat.0) < continued_fraction_real_limit(
        sqrt_two_continued_fraction_coefficients) and
    continued_fraction_real_limit(sqrt_two_continued_fraction_coefficients) < continued_fraction_real_convergent_value(
        sqrt_two_continued_fraction_coefficients, Nat.1)
} by {
    continued_fraction_properties_sqrt_two_positive_tail
    positive_continued_fraction_sequence_tail(
        sqrt_two_continued_fraction_coefficients)
    continued_fraction_real_even_lt_limit(
        sqrt_two_continued_fraction_coefficients, Nat.0)
    continued_fraction_real_even_convergent(
            sqrt_two_continued_fraction_coefficients, Nat.0) < continued_fraction_real_limit(
        sqrt_two_continued_fraction_coefficients)
    continued_fraction_real_even_convergent(
            sqrt_two_continued_fraction_coefficients, Nat.0) =
        continued_fraction_real_convergent_value(
            sqrt_two_continued_fraction_coefficients, Nat.2 * Nat.0)
    Nat.2 * Nat.0 = Nat.0
    continued_fraction_real_convergent_value(
            sqrt_two_continued_fraction_coefficients, Nat.0) < continued_fraction_real_limit(
        sqrt_two_continued_fraction_coefficients)
    continued_fraction_real_limit_lt_odd(
        sqrt_two_continued_fraction_coefficients, Nat.0)
    continued_fraction_real_limit(sqrt_two_continued_fraction_coefficients) < continued_fraction_real_odd_convergent(
        sqrt_two_continued_fraction_coefficients, Nat.0)
    continued_fraction_real_odd_convergent(
            sqrt_two_continued_fraction_coefficients, Nat.0) =
        continued_fraction_real_convergent_value(
            sqrt_two_continued_fraction_coefficients, (Nat.2 * Nat.0).suc)
    Nat.2 * Nat.0 = Nat.0
    (Nat.2 * Nat.0).suc = Nat.1
    continued_fraction_real_limit(sqrt_two_continued_fraction_coefficients) < continued_fraction_real_convergent_value(
        sqrt_two_continued_fraction_coefficients, Nat.1)
    continued_fraction_real_convergent_value(
            sqrt_two_continued_fraction_coefficients, Nat.0) < continued_fraction_real_limit(
        sqrt_two_continued_fraction_coefficients) and
    continued_fraction_real_limit(sqrt_two_continued_fraction_coefficients) < continued_fraction_real_convergent_value(
        sqrt_two_continued_fraction_coefficients, Nat.1)
}

// ============================================================================
// Section 2: the Pell norm of a convergent of √2 is ±1
// ============================================================================

/// Every convergent of √2 has Pell norm one or minus one:
/// p_n² - 2·q_n² = ±1.
theorem continued_fraction_properties_sqrt_two_norm_one_or_neg_one(n: Nat) {
    pell_norm(Int.2,
        Int.from_nat(continued_fraction_convergent_numerator(
            sqrt_two_continued_fraction_coefficients, n)),
        Int.from_nat(continued_fraction_convergent_denominator(
            sqrt_two_continued_fraction_coefficients, n))) = Int.1 or
    pell_norm(Int.2,
        Int.from_nat(continued_fraction_convergent_numerator(
            sqrt_two_continued_fraction_coefficients, n)),
        Int.from_nat(continued_fraction_convergent_denominator(
            sqrt_two_continued_fraction_coefficients, n))) = -Int.1
} by {
    cf_pell_convergent_norm_one_or_neg_one(n)
    pell_norm(Int.2,
        Int.from_nat(continued_fraction_convergent_numerator(
            sqrt_two_continued_fraction_coefficients, n)),
        Int.from_nat(continued_fraction_convergent_denominator(
            sqrt_two_continued_fraction_coefficients, n))) = Int.1 or
    pell_norm(Int.2,
        Int.from_nat(continued_fraction_convergent_numerator(
            sqrt_two_continued_fraction_coefficients, n)),
        Int.from_nat(continued_fraction_convergent_denominator(
            sqrt_two_continued_fraction_coefficients, n))) = -Int.1
}

/// The integer value p_n² - 2·q_n² of a convergent of √2.
define sqrt_two_convergent_norm_int(n: Nat) -> Int {
    Int.from_nat(continued_fraction_convergent_numerator(
            sqrt_two_continued_fraction_coefficients, n)) *
        Int.from_nat(continued_fraction_convergent_numerator(
            sqrt_two_continued_fraction_coefficients, n)) -
        Int.2 * (Int.from_nat(continued_fraction_convergent_denominator(
                sqrt_two_continued_fraction_coefficients, n)) *
            Int.from_nat(continued_fraction_convergent_denominator(
                sqrt_two_continued_fraction_coefficients, n)))
}

/// The Pell norm of a convergent of √2 is its classical norm p_n² - 2·q_n².
theorem continued_fraction_properties_sqrt_two_norm_unfolded_eq(n: Nat) {
    pell_norm(Int.2,
        Int.from_nat(continued_fraction_convergent_numerator(
            sqrt_two_continued_fraction_coefficients, n)),
        Int.from_nat(continued_fraction_convergent_denominator(
            sqrt_two_continued_fraction_coefficients, n))) =
        sqrt_two_convergent_norm_int(n)
} by {
    pell_norm(Int.2,
        Int.from_nat(continued_fraction_convergent_numerator(
            sqrt_two_continued_fraction_coefficients, n)),
        Int.from_nat(continued_fraction_convergent_denominator(
            sqrt_two_continued_fraction_coefficients, n))) =
        sqrt_two_convergent_norm_int(n)
}

/// Unfolding the Pell norm, the classical identity p_n² - 2·q_n² = ±1 holds
/// for every convergent of √2.
theorem continued_fraction_properties_sqrt_two_norm_unfolded_one_or_neg_one(n: Nat) {
    sqrt_two_convergent_norm_int(n) = Int.1 or
        sqrt_two_convergent_norm_int(n) = -Int.1
} by {
    continued_fraction_properties_sqrt_two_norm_one_or_neg_one(n)
    pell_norm(Int.2,
        Int.from_nat(continued_fraction_convergent_numerator(
            sqrt_two_continued_fraction_coefficients, n)),
        Int.from_nat(continued_fraction_convergent_denominator(
            sqrt_two_continued_fraction_coefficients, n))) = Int.1 or
    pell_norm(Int.2,
        Int.from_nat(continued_fraction_convergent_numerator(
            sqrt_two_continued_fraction_coefficients, n)),
        Int.from_nat(continued_fraction_convergent_denominator(
            sqrt_two_continued_fraction_coefficients, n))) = -Int.1
    continued_fraction_properties_sqrt_two_norm_unfolded_eq(n)
    pell_norm(Int.2,
        Int.from_nat(continued_fraction_convergent_numerator(
            sqrt_two_continued_fraction_coefficients, n)),
        Int.from_nat(continued_fraction_convergent_denominator(
            sqrt_two_continued_fraction_coefficients, n))) =
        sqrt_two_convergent_norm_int(n)
    if pell_norm(Int.2,
            Int.from_nat(continued_fraction_convergent_numerator(
                sqrt_two_continued_fraction_coefficients, n)),
            Int.from_nat(continued_fraction_convergent_denominator(
                sqrt_two_continued_fraction_coefficients, n))) = Int.1 {
        sqrt_two_convergent_norm_int(n) = Int.1
        sqrt_two_convergent_norm_int(n) = Int.1 or
            sqrt_two_convergent_norm_int(n) = -Int.1
    }
    if pell_norm(Int.2,
            Int.from_nat(continued_fraction_convergent_numerator(
                sqrt_two_continued_fraction_coefficients, n)),
            Int.from_nat(continued_fraction_convergent_denominator(
                sqrt_two_continued_fraction_coefficients, n))) = -Int.1 {
        sqrt_two_convergent_norm_int(n) = -Int.1
        sqrt_two_convergent_norm_int(n) = Int.1 or
            sqrt_two_convergent_norm_int(n) = -Int.1
    }
    sqrt_two_convergent_norm_int(n) = Int.1 or
        sqrt_two_convergent_norm_int(n) = -Int.1
}

/// The absolute value of the Pell norm of a convergent of √2 is one:
/// |p_n² - 2·q_n²| = 1.
theorem continued_fraction_properties_sqrt_two_norm_abs_one(n: Nat) {
    abs(pell_norm(Int.2,
        Int.from_nat(continued_fraction_convergent_numerator(
            sqrt_two_continued_fraction_coefficients, n)),
        Int.from_nat(continued_fraction_convergent_denominator(
            sqrt_two_continued_fraction_coefficients, n)))) = Nat.1
} by {
    cf_pell_convergent_norm_abs_one(n)
    abs(pell_norm(Int.2,
        Int.from_nat(continued_fraction_convergent_numerator(
            sqrt_two_continued_fraction_coefficients, n)),
        Int.from_nat(continued_fraction_convergent_denominator(
            sqrt_two_continued_fraction_coefficients, n)))) = Nat.1
}

// ============================================================================
// Section 3: the determinant identity p_n·q_{n-1} - p_{n-1}·q_n = ±1
// ============================================================================

/// The signed determinant of adjacent convergents is the alternating sign:
/// p_{n+1}·q_n - p_n·q_{n+1} = (-1)^n.
theorem continued_fraction_properties_adjacent_determinant_alternating(
    coefficients: Nat -> Nat, n: Nat
) {
    Int.from_nat(continued_fraction_convergent_numerator(coefficients, n.suc)) *
            Int.from_nat(continued_fraction_convergent_denominator(
                coefficients, n)) -
        Int.from_nat(continued_fraction_convergent_numerator(
            coefficients, n)) *
            Int.from_nat(continued_fraction_convergent_denominator(
                coefficients, n.suc)) =
        alternating_sign[Int](n)
} by {
    continued_fraction_adjacent_convergent_determinant_identity(coefficients, n)
    continued_fraction_adjacent_convergent_determinant(coefficients, n) =
        alternating_sign[Int](n)
    continued_fraction_adjacent_convergent_determinant(coefficients, n) =
        Int.from_nat(continued_fraction_convergent_numerator(coefficients, n.suc)) *
            Int.from_nat(continued_fraction_convergent_denominator(
                coefficients, n)) -
        Int.from_nat(continued_fraction_convergent_numerator(coefficients, n)) *
            Int.from_nat(continued_fraction_convergent_denominator(
                coefficients, n.suc))
    Int.from_nat(continued_fraction_convergent_numerator(coefficients, n.suc)) *
            Int.from_nat(continued_fraction_convergent_denominator(
                coefficients, n)) -
        Int.from_nat(continued_fraction_convergent_numerator(coefficients, n)) *
            Int.from_nat(continued_fraction_convergent_denominator(
                coefficients, n.suc)) =
        alternating_sign[Int](n)
}

/// The determinant identity in its classical form: adjacent convergents
/// satisfy p_n·q_{n-1} - p_{n-1}·q_n = ±1 (stated at every index as
/// p_{n+1}·q_n - p_n·q_{n+1} = ±1).
theorem continued_fraction_properties_adjacent_determinant_one_or_neg_one(
    coefficients: Nat -> Nat, n: Nat
) {
    Int.from_nat(continued_fraction_convergent_numerator(coefficients, n.suc)) *
            Int.from_nat(continued_fraction_convergent_denominator(
                coefficients, n)) -
        Int.from_nat(continued_fraction_convergent_numerator(coefficients, n)) *
            Int.from_nat(continued_fraction_convergent_denominator(
                coefficients, n.suc)) = Int.1 or
    Int.from_nat(continued_fraction_convergent_numerator(coefficients, n.suc)) *
            Int.from_nat(continued_fraction_convergent_denominator(
                coefficients, n)) -
        Int.from_nat(continued_fraction_convergent_numerator(coefficients, n)) *
            Int.from_nat(continued_fraction_convergent_denominator(
                coefficients, n.suc)) = -Int.1
} by {
    continued_fraction_properties_adjacent_determinant_alternating(coefficients, n)
    Int.from_nat(continued_fraction_convergent_numerator(coefficients, n.suc)) *
            Int.from_nat(continued_fraction_convergent_denominator(
                coefficients, n)) -
        Int.from_nat(continued_fraction_convergent_numerator(coefficients, n)) *
            Int.from_nat(continued_fraction_convergent_denominator(
                coefficients, n.suc)) =
        alternating_sign[Int](n)
    continued_fraction_alternating_sign_one_or_neg_one(n)
    alternating_sign[Int](n) = Int.1 or alternating_sign[Int](n) = -Int.1
    if alternating_sign[Int](n) = Int.1 {
        Int.from_nat(continued_fraction_convergent_numerator(coefficients, n.suc)) *
                Int.from_nat(continued_fraction_convergent_denominator(
                    coefficients, n)) -
            Int.from_nat(continued_fraction_convergent_numerator(coefficients, n)) *
                Int.from_nat(continued_fraction_convergent_denominator(
                    coefficients, n.suc)) = Int.1
        Int.from_nat(continued_fraction_convergent_numerator(coefficients, n.suc)) *
                Int.from_nat(continued_fraction_convergent_denominator(
                    coefficients, n)) -
            Int.from_nat(continued_fraction_convergent_numerator(coefficients, n)) *
                Int.from_nat(continued_fraction_convergent_denominator(
                    coefficients, n.suc)) = Int.1 or
        Int.from_nat(continued_fraction_convergent_numerator(coefficients, n.suc)) *
                Int.from_nat(continued_fraction_convergent_denominator(
                    coefficients, n)) -
            Int.from_nat(continued_fraction_convergent_numerator(coefficients, n)) *
                Int.from_nat(continued_fraction_convergent_denominator(
                    coefficients, n.suc)) = -Int.1
    }
    if alternating_sign[Int](n) = -Int.1 {
        Int.from_nat(continued_fraction_convergent_numerator(coefficients, n.suc)) *
                Int.from_nat(continued_fraction_convergent_denominator(
                    coefficients, n)) -
            Int.from_nat(continued_fraction_convergent_numerator(coefficients, n)) *
                Int.from_nat(continued_fraction_convergent_denominator(
                    coefficients, n.suc)) = -Int.1
        Int.from_nat(continued_fraction_convergent_numerator(coefficients, n.suc)) *
                Int.from_nat(continued_fraction_convergent_denominator(
                    coefficients, n)) -
            Int.from_nat(continued_fraction_convergent_numerator(coefficients, n)) *
                Int.from_nat(continued_fraction_convergent_denominator(
                    coefficients, n.suc)) = Int.1 or
        Int.from_nat(continued_fraction_convergent_numerator(coefficients, n.suc)) *
                Int.from_nat(continued_fraction_convergent_denominator(
                    coefficients, n)) -
            Int.from_nat(continued_fraction_convergent_numerator(coefficients, n)) *
                Int.from_nat(continued_fraction_convergent_denominator(
                    coefficients, n.suc)) = -Int.1
    }
    Int.from_nat(continued_fraction_convergent_numerator(coefficients, n.suc)) *
            Int.from_nat(continued_fraction_convergent_denominator(
                coefficients, n)) -
        Int.from_nat(continued_fraction_convergent_numerator(coefficients, n)) *
            Int.from_nat(continued_fraction_convergent_denominator(
                coefficients, n.suc)) = Int.1 or
    Int.from_nat(continued_fraction_convergent_numerator(coefficients, n.suc)) *
            Int.from_nat(continued_fraction_convergent_denominator(
                coefficients, n)) -
        Int.from_nat(continued_fraction_convergent_numerator(coefficients, n)) *
            Int.from_nat(continued_fraction_convergent_denominator(
                coefficients, n.suc)) = -Int.1
}

/// The absolute value of the adjacent-convergent determinant is one:
/// |p_n·q_{n-1} - p_{n-1}·q_n| = 1.
theorem continued_fraction_properties_adjacent_determinant_abs_one(
    coefficients: Nat -> Nat, n: Nat
) {
    abs(Int.from_nat(continued_fraction_convergent_numerator(coefficients, n.suc)) *
            Int.from_nat(continued_fraction_convergent_denominator(
                coefficients, n)) -
        Int.from_nat(continued_fraction_convergent_numerator(coefficients, n)) *
            Int.from_nat(continued_fraction_convergent_denominator(
                coefficients, n.suc))) =
        Nat.1
} by {
    continued_fraction_convergent_determinant_abs_one(coefficients, n)
    abs(Int.from_nat(continued_fraction_convergent_numerator(coefficients, n)) *
            Int.from_nat(continued_fraction_convergent_denominator(
                coefficients, n.suc)) -
        Int.from_nat(continued_fraction_convergent_numerator(coefficients, n.suc)) *
            Int.from_nat(continued_fraction_convergent_denominator(
                coefficients, n))) =
        Nat.1
    neg_sub(
        Int.from_nat(continued_fraction_convergent_numerator(coefficients, n)) *
            Int.from_nat(continued_fraction_convergent_denominator(
                coefficients, n.suc)),
        Int.from_nat(continued_fraction_convergent_numerator(coefficients, n.suc)) *
            Int.from_nat(continued_fraction_convergent_denominator(
                coefficients, n)))
    Int.from_nat(continued_fraction_convergent_numerator(coefficients, n.suc)) *
            Int.from_nat(continued_fraction_convergent_denominator(
                coefficients, n)) -
        Int.from_nat(continued_fraction_convergent_numerator(coefficients, n)) *
            Int.from_nat(continued_fraction_convergent_denominator(
                coefficients, n.suc)) =
        -(Int.from_nat(continued_fraction_convergent_numerator(coefficients, n)) *
                Int.from_nat(continued_fraction_convergent_denominator(
                    coefficients, n.suc)) -
            Int.from_nat(continued_fraction_convergent_numerator(coefficients, n.suc)) *
                Int.from_nat(continued_fraction_convergent_denominator(
                    coefficients, n)))
    abs_neg(
        Int.from_nat(continued_fraction_convergent_numerator(coefficients, n)) *
            Int.from_nat(continued_fraction_convergent_denominator(
                coefficients, n.suc)) -
        Int.from_nat(continued_fraction_convergent_numerator(coefficients, n.suc)) *
            Int.from_nat(continued_fraction_convergent_denominator(
                coefficients, n)))
    abs(-(Int.from_nat(continued_fraction_convergent_numerator(coefficients, n)) *
                Int.from_nat(continued_fraction_convergent_denominator(
                    coefficients, n.suc)) -
            Int.from_nat(continued_fraction_convergent_numerator(coefficients, n.suc)) *
                Int.from_nat(continued_fraction_convergent_denominator(
                    coefficients, n)))) =
        abs(Int.from_nat(continued_fraction_convergent_numerator(coefficients, n)) *
                Int.from_nat(continued_fraction_convergent_denominator(
                    coefficients, n.suc)) -
            Int.from_nat(continued_fraction_convergent_numerator(coefficients, n.suc)) *
                Int.from_nat(continued_fraction_convergent_denominator(
                    coefficients, n)))
    abs(Int.from_nat(continued_fraction_convergent_numerator(coefficients, n.suc)) *
            Int.from_nat(continued_fraction_convergent_denominator(
                coefficients, n)) -
        Int.from_nat(continued_fraction_convergent_numerator(coefficients, n)) *
            Int.from_nat(continued_fraction_convergent_denominator(
                coefficients, n.suc))) =
        Nat.1
}

// ============================================================================
// Section 4: the irrationality of √2
// ============================================================================

/// True when a natural number is a perfect square.
define is_perfect_square(n: Nat) -> Bool {
    exists(m: Nat) { n = m * m }
}

/// The Diophantine equation x² = 2·y² has no solution with nonzero
/// denominator: √2 is irrational.
theorem continued_fraction_properties_sqrt_two_irrational_nat(p: Nat, q: Nat) {
    p * p = Nat.2 * (q * q) implies q = Nat.0
} by {
    if p * p = Nat.2 * (q * q) {
        no_nontrivial_sq_eq_two_sq(p, q)
        p = Nat.0 and q = Nat.0
        q = Nat.0
    }
}

/// No positive rational squares to two: √2 is irrational.
theorem continued_fraction_properties_sqrt_two_no_positive_rational(
    p: Nat, q: Nat
) {
    Nat.0 < q implies p * p != Nat.2 * (q * q)
} by {
    if Nat.0 < q {
        if p * p = Nat.2 * (q * q) {
            no_nontrivial_sq_eq_two_sq(p, q)
            p = Nat.0 and q = Nat.0
            q = Nat.0
            lt_not_ref(Nat.0)
            false
        }
        p * p != Nat.2 * (q * q)
    }
}

/// The Diophantine equation x² = 2·y² has no integer solution with nonzero
/// denominator: √2 is irrational over the integers as well.
theorem continued_fraction_properties_sqrt_two_irrational_int(x: Int, y: Int) {
    x * x = Int.2 * (y * y) implies y = Int.0
} by {
    if x * x = Int.2 * (y * y) {
        no_nontrivial_int_sq_eq_two_sq(x, y)
        x = Int.0 and y = Int.0
        y = Int.0
    }
}

// The general statement — √N is irrational unless N is a perfect square —
// requires the prime-factorization form of "not a perfect square".  In the
// Diophantine encoding x² = N·y² it reads:
//
// theorem continued_fraction_properties_sqrt_n_irrational_unless_square(n: Nat) {
//     not is_perfect_square(n) implies forall(p: Nat, q: Nat) {
//         p * p = n * (q * q) implies q = Nat.0
//     }
// }
//
// For n = 2 the statement is proved above
// (continued_fraction_properties_sqrt_two_irrational_nat), since two is not
// a perfect square and the case analysis on the parity of the exponent of
// two in the prime factorization is handled by the descent in diophantine.ac.

// ============================================================================
// Section 5: the best-approximation property of convergents
// ============================================================================

// The classical best-approximation theorem states that every convergent is a
// best approximation of the value α of the continued fraction: for every
// rational p/q distinct from p_n/q_n with denominator q < q_{n+1},
//
//     |α - p_n/q_n| < |α - p/q|.
//
// The library contains the two ingredients of the standard proof — the
// approximation estimate |α - p_n/q_n| < 1/q_n²
// (continued_fraction_approximation_estimate) and the denominator bound for
// fractions strictly between adjacent convergents
// (continued_fraction_between_adjacent_denominator_bound_of_positive_sign and
// continued_fraction_between_adjacent_denominator_bound_of_negative_sign) —
// but the full statement is left for future work.  In the library's
// formulation with α = continued_fraction_real_limit(coefficients) it would
// read:
//
// theorem continued_fraction_properties_best_approximation(
//     coefficients: Nat -> Nat, n: Nat, p: Nat, q: Nat
// ) {
//     positive_continued_fraction_sequence_tail(coefficients) and
//         q < continued_fraction_convergent_denominator(coefficients, n.suc) and
//         Rat.from_nat(p) / Rat.from_nat(q) !=
//             continued_fraction_convergent_value(coefficients, n) implies
//         (continued_fraction_real_limit(coefficients) -
//             continued_fraction_real_convergent_value(coefficients, n)).abs <
//         (continued_fraction_real_limit(coefficients) -
//             Real.from_rat(Rat.from_nat(p) / Rat.from_nat(q))).abs
// }
