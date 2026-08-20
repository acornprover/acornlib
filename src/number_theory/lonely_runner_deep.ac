/// The lonely runner conjecture, deepened.
///
/// Runner 0 is the lonely runner, stationary at the origin; the remaining runners
/// move at distinct positive integer speeds.  With `n + 1` runners in total, the
/// lonely condition requires every moving runner to be at circular distance at
/// least `1 / (n + 2)` from the origin.  This file:
///
///   1. States the lonely-runner view: the condition depends only on the
///      fractional parts `frac(v_i * t)` of the positions, and a circular
///      distance is at least `b` exactly when the fractional part lies in the
///      interval `[b, 1 - b]` (Section 1).  The equivalence is proved for each
///      runner of the two- and three-runner cases, and the fractional-part
///      witnesses are exhibited.
///
///   2. Verifies the small cases with speeds `0, 1` (two runners, Section 2a)
///      and `0, 1, 2` (three runners, Section 2b), confirming the existing
///      work in `analysis/real/lonely_runner.ac`: at `t = 1/2` the moving
///      runner is at distance `1/2 >= 1/3`, and at `t = 1/4` the runners are at
///      `1/4` and `1/2`, both at distance at least `1/4`.
///
///   3. Proves the equispaced case (Section 3): runners with speeds
///      `0, 1, 2, ..., n` (that is, `n + 1` runners) are all lonely at the
///      single time `t = 1 / (n + 2)`, when every runner sits at the lattice
///      point `i / (n + 2)` at distance at least `1 / (n + 2)` from the origin.
///      This is the arithmetic-progression case of the conjecture, generalizing
///      the concrete witnesses `t = 1/4`, `t = 1/5`, `t = 1/6` of the small
///      cases.  The half time `t = 1 / (2 (n + 1))` does not work: the adjacent
///      runners are then exactly `1 / (2 (n + 1))` apart, below `1 / (n + 1)`.
///
/// The full conjecture (Section 4) is an open problem in general (it is known
/// for up to seven moving runners); it is stated below but not proved here.
from nat import Nat, from_nat, from_nat_add, from_nat_mul, from_nat_one, from_nat_zero,
    lt_add_suc, lt_imp_lte_suc, lte_and_lt, lt_and_lte, add_comm, lt_or_lte,
    not_lt_zero, add_imp_sub, lte_add_left
from int import Int
from order import lt_imp_lte, lt_of_lt_of_lte, lte_trans, lte_min_iff, lte_ref
from real import Real, floor, lt_add_one, lt_lte_trans, lte_lt_trans, mul_one_left,
    mul_one_over, mul_div_cancel, div_cancel_common, sub_cancels, mul_sub_distrib_left,
    mul_inverse, lt_add_right, lt_add_left, lte_add_right, add_from_int
from algebra.add_ordered_group import add_le_iff_le_sub_right
from ordered_field import mul_lt_mul_of_pos_right, inverse_of_positive_is_positive,
    inverse_on_positive_flips_inequality
from algebra.field.field import mul_not_zero
from analysis import real_from_int_lte, floor_is_greatest, int_lt_imp_add_one_lte,
    from_nat_real_lte, div_le_div_pos, from_nat_real_nonnegative, frac,
    circular_distance, cd_bound, cd_bound_frac, floor_zero_of_unit_interval,
    frac_of_unit_interval, circular_distance_of_unit_interval, from_nat_real_positive,
    same_denom_lte, from_nat_frac_lt_one, one_minus_frac, from_nat_real_lt,
    lonely_runner_two, nat_one_le_one, nat_lt_one_two, nat_lt_one_three,
    nat_lt_one_four, nat_lt_two_four, nat_one_le_two, nat_one_le_three
from data.basic.logic import exists_intro

numerals Real
numerals Nat

// ============================================================================
// Section 1: the lonely-runner view (fractional parts)
// ============================================================================

/// Zero is at most every natural.
theorem nat_zero_lte(n: Nat) {
    Nat.0 <= n
} by {
    lt_or_lte(n, Nat.0)
    n < Nat.0 or Nat.0 <= n
    not_lt_zero(n)
    not (n < Nat.0)
    Nat.0 <= n
}

/// One is below `n + 2`.
theorem nat_one_lt_add_two(n: Nat) {
    Nat.1 < n + Nat.2
} by {
    lt_add_suc(Nat.1, Nat.0)
    (Nat.1 < Nat.1 + Nat.0.suc)
    (Nat.1 + Nat.0.suc = Nat.2)
    Nat.1 < Nat.2
    nat_zero_lte(n)
    Nat.0 <= n
    lte_add_left(Nat.2, Nat.0, n)
    (Nat.2 + Nat.0 <= Nat.2 + n)
    (Nat.2 + Nat.0 = Nat.2)
    (Nat.2 + n = n + Nat.2)
    Nat.2 <= n + Nat.2
    lt_of_lt_of_lte(Nat.1, Nat.2, n + Nat.2)
    Nat.1 < n + Nat.2
}

/// Subtracting one from `n + 2` leaves `n + 1`.
theorem nat_add_two_sub_one(n: Nat) {
    n + Nat.2 - Nat.1 = n + Nat.1
} by {
    add_imp_sub(n + Nat.1, Nat.1, n + Nat.2)
    ((n + Nat.1) + Nat.1 = n + Nat.2)
    n + Nat.2 - Nat.1 = n + Nat.1
}

/// The fractional part of a real is nonnegative.
theorem frac_nonneg(x: Real) {
    Real.0 <= frac(x)
} by {
    (Real.from_int(floor(x)) <= x)
    lte_add_right(Real.from_int(floor(x)), x, -Real.from_int(floor(x)))
    (Real.from_int(floor(x)) + -Real.from_int(floor(x)) <= x + -Real.from_int(floor(x)))
    (Real.from_int(floor(x)) + -Real.from_int(floor(x)) = Real.0)
    (x + -Real.from_int(floor(x)) = x - Real.from_int(floor(x)))
    (Real.0 <= x - Real.from_int(floor(x)))
    frac(x) = x - Real.from_int(floor(x))
    Real.0 <= frac(x)
}

/// The fractional part of a real is below one.
theorem frac_lt_one(x: Real) {
    frac(x) < Real.1
} by {
    (x < Real.from_int(floor(x) + Int.1))
    add_from_int(floor(x), Int.1)
    (Real.from_int(floor(x)) + Real.from_int(Int.1) = Real.from_int(floor(x) + Int.1))
    (Real.from_int(Int.1) = Real.1)
    (Real.from_int(floor(x)) + Real.1 = Real.from_int(floor(x) + Int.1))
    (x < Real.from_int(floor(x)) + Real.1)
    lt_add_right(x, Real.from_int(floor(x)) + Real.1, -Real.from_int(floor(x)))
    (x + -Real.from_int(floor(x)) < Real.from_int(floor(x)) + Real.1 + -Real.from_int(floor(x)))
    (Real.from_int(floor(x)) + Real.1 + -Real.from_int(floor(x))
        = Real.from_int(floor(x)) + Real.1 - Real.from_int(floor(x)))
    sub_cancels(Real.from_int(floor(x)), Real.1)
    (Real.from_int(floor(x)) + Real.1 - Real.from_int(floor(x)) = Real.1)
    (Real.from_int(floor(x)) + Real.1 + -Real.from_int(floor(x)) = Real.1)
    (x - Real.from_int(floor(x)) < Real.1)
    frac(x) = x - Real.from_int(floor(x))
    frac(x) < Real.1
}

/// The fractional part of a fractional part is the fractional part itself.
theorem frac_of_frac(x: Real) {
    frac(frac(x)) = frac(x)
} by {
    frac(frac(x)) = frac(x) - Real.from_int(floor(frac(x)))
    frac_nonneg(x)
    (Real.0 <= frac(x))
    frac_lt_one(x)
    (frac(x) < Real.1)
    (Real.0 <= frac(x) and frac(x) < Real.1)
    floor_zero_of_unit_interval(frac(x))
    floor(frac(x)) = Int.0
    (Real.from_int(Int.0) = Real.0)
    (frac(x) - Real.from_int(floor(frac(x))) = frac(x))
    frac(frac(x)) = frac(x)
}

/// The circular distance depends only on the fractional part.
///
/// The heart of the lonely-runner view: the position of a runner enters the
/// lonely condition only through its fractional part.
theorem circular_distance_of_frac(x: Real) {
    circular_distance(x) = circular_distance(frac(x))
} by {
    circular_distance(x) = frac(x).min(Real.1 - frac(x))
    circular_distance(frac(x)) = frac(frac(x)).min(Real.1 - frac(frac(x)))
    frac_of_frac(x)
    frac(frac(x)) = frac(x)
    circular_distance(frac(x)) = frac(x).min(Real.1 - frac(x))
    circular_distance(x) = circular_distance(frac(x))
}

/// A circular distance is at least `b` exactly when the fractional part lies
/// in the interval `[b, 1 - b]`.
///
/// The reformulation of the lonely condition in terms of fractional parts: a
/// runner is lonely at time `t` exactly when the fractional part of its
/// position stays away from both ends of the unit interval.
theorem cd_iff_frac_interval(x: Real, b: Real) {
    b <= circular_distance(x) = (b <= frac(x) and frac(x) <= Real.1 - b)
} by {
    circular_distance(x) = frac(x).min(Real.1 - frac(x))
    lte_min_iff(b, frac(x), Real.1 - frac(x))
    (b <= frac(x).min(Real.1 - frac(x)) = (b <= frac(x) and b <= Real.1 - frac(x)))
    add_le_iff_le_sub_right(b, frac(x), Real.1)
    (b + frac(x) <= Real.1 = (b <= Real.1 - frac(x)))
    (b + frac(x) = frac(x) + b)
    (frac(x) + b <= Real.1 = (b <= Real.1 - frac(x)))
    add_le_iff_le_sub_right(frac(x), b, Real.1)
    (frac(x) + b <= Real.1 = (frac(x) <= Real.1 - b))
    (b <= Real.1 - frac(x) = (frac(x) <= Real.1 - b))
    (b <= frac(x) and b <= Real.1 - frac(x)) = (b <= frac(x) and frac(x) <= Real.1 - b)
    b <= circular_distance(x) = (b <= frac(x) and frac(x) <= Real.1 - b)
}

// ----------------------------------------------------------------------------
// The equivalence for the small cases
// ----------------------------------------------------------------------------

/// For two runners (speeds 0 and 1), the lonely condition at time `t` is
/// exactly that the fractional part of `t` lies in `[1/3, 2/3]`.
theorem lonely_runner_two_frac_view(t: Real) {
    (Real.1 / from_nat[Real](Nat.3) <= circular_distance(Real.1 * t)
        = (Real.1 / from_nat[Real](Nat.3) <= frac(Real.1 * t)
            and frac(Real.1 * t) <= from_nat[Real](Nat.2) / from_nat[Real](Nat.3)))
} by {
    cd_iff_frac_interval(Real.1 * t, Real.1 / from_nat[Real](Nat.3))
    (Real.1 / from_nat[Real](Nat.3) <= circular_distance(Real.1 * t)
        = (Real.1 / from_nat[Real](Nat.3) <= frac(Real.1 * t)
            and frac(Real.1 * t) <= Real.1 - Real.1 / from_nat[Real](Nat.3)))
    one_minus_frac(Nat.1, Nat.3)
    (Real.1 - from_nat[Real](Nat.1) / from_nat[Real](Nat.3)
        = from_nat[Real](Nat.3 - Nat.1) / from_nat[Real](Nat.3))
    (from_nat[Real](Nat.1) = Real.1)
    (Real.1 - Real.1 / from_nat[Real](Nat.3)
        = from_nat[Real](Nat.2) / from_nat[Real](Nat.3))
    (Real.1 / from_nat[Real](Nat.3) <= circular_distance(Real.1 * t)
        = (Real.1 / from_nat[Real](Nat.3) <= frac(Real.1 * t)
            and frac(Real.1 * t) <= from_nat[Real](Nat.2) / from_nat[Real](Nat.3)))
}

/// For three runners (speeds 0, 1 and 2), the lonely condition for runner 1
/// (speed 1) at time `t` is exactly that the fractional part of `t` lies in
/// `[1/4, 3/4]`.
theorem lonely_runner_three_frac_view_one(t: Real) {
    (Real.1 / from_nat[Real](Nat.4) <= circular_distance(Real.1 * t)
        = (Real.1 / from_nat[Real](Nat.4) <= frac(Real.1 * t)
            and frac(Real.1 * t) <= from_nat[Real](Nat.3) / from_nat[Real](Nat.4)))
} by {
    cd_iff_frac_interval(Real.1 * t, Real.1 / from_nat[Real](Nat.4))
    (Real.1 / from_nat[Real](Nat.4) <= circular_distance(Real.1 * t)
        = (Real.1 / from_nat[Real](Nat.4) <= frac(Real.1 * t)
            and frac(Real.1 * t) <= Real.1 - Real.1 / from_nat[Real](Nat.4)))
    one_minus_frac(Nat.1, Nat.4)
    (Real.1 - from_nat[Real](Nat.1) / from_nat[Real](Nat.4)
        = from_nat[Real](Nat.4 - Nat.1) / from_nat[Real](Nat.4))
    (from_nat[Real](Nat.1) = Real.1)
    (Real.1 - Real.1 / from_nat[Real](Nat.4)
        = from_nat[Real](Nat.3) / from_nat[Real](Nat.4))
    (Real.1 / from_nat[Real](Nat.4) <= circular_distance(Real.1 * t)
        = (Real.1 / from_nat[Real](Nat.4) <= frac(Real.1 * t)
            and frac(Real.1 * t) <= from_nat[Real](Nat.3) / from_nat[Real](Nat.4)))
}

/// For three runners (speeds 0, 1 and 2), the lonely condition for runner 2
/// (speed 2) at time `t` is exactly that the fractional part of `2t` lies in
/// `[1/4, 3/4]`.
theorem lonely_runner_three_frac_view_two(t: Real) {
    (Real.1 / from_nat[Real](Nat.4) <= circular_distance(from_nat[Real](Nat.2) * t)
        = (Real.1 / from_nat[Real](Nat.4) <= frac(from_nat[Real](Nat.2) * t)
            and frac(from_nat[Real](Nat.2) * t) <= from_nat[Real](Nat.3) / from_nat[Real](Nat.4)))
} by {
    cd_iff_frac_interval(from_nat[Real](Nat.2) * t, Real.1 / from_nat[Real](Nat.4))
    (Real.1 / from_nat[Real](Nat.4) <= circular_distance(from_nat[Real](Nat.2) * t)
        = (Real.1 / from_nat[Real](Nat.4) <= frac(from_nat[Real](Nat.2) * t)
            and frac(from_nat[Real](Nat.2) * t) <= Real.1 - Real.1 / from_nat[Real](Nat.4)))
    one_minus_frac(Nat.1, Nat.4)
    (Real.1 - from_nat[Real](Nat.1) / from_nat[Real](Nat.4)
        = from_nat[Real](Nat.4 - Nat.1) / from_nat[Real](Nat.4))
    (from_nat[Real](Nat.1) = Real.1)
    (Real.1 - Real.1 / from_nat[Real](Nat.4)
        = from_nat[Real](Nat.3) / from_nat[Real](Nat.4))
    (Real.1 / from_nat[Real](Nat.4) <= circular_distance(from_nat[Real](Nat.2) * t)
        = (Real.1 / from_nat[Real](Nat.4) <= frac(from_nat[Real](Nat.2) * t)
            and frac(from_nat[Real](Nat.2) * t) <= from_nat[Real](Nat.3) / from_nat[Real](Nat.4)))
}

// ============================================================================
// Section 2: the small cases
// ============================================================================

// ----------------------------------------------------------------------------
// Section 2a: two runners, speeds 0 and 1
// ----------------------------------------------------------------------------

/// Two runners (speeds 0 and 1): at time 1/2 the moving runner is at circular
/// distance 1/2 >= 1/3 from the origin, so both runners are lonely at time 1/2
/// (each sees the other at distance 1/2).  This is the n = 2 case, threshold
/// 1/3, witnessed here at the half time.
theorem lonely_runner_two_speeds_01 {
    exists(t: Real) {
        Real.1 / from_nat[Real](Nat.3) <= circular_distance(Real.1 * t)
    }
} by {
    mul_one_left(Real.1 / from_nat[Real](Nat.2))
    (Real.1 * (Real.1 / from_nat[Real](Nat.2)) = Real.1 / from_nat[Real](Nat.2))
    cd_bound_frac(Nat.1, Nat.2)
    (Nat.1 <= Nat.1 and Nat.1 < Nat.2)
    (Real.1 / from_nat[Real](Nat.2) <= circular_distance(from_nat[Real](Nat.1) / from_nat[Real](Nat.2)))
    (from_nat[Real](Nat.1) = Real.1)
    (Real.1 / from_nat[Real](Nat.2) <= circular_distance(Real.1 / from_nat[Real](Nat.2)))
    (Real.1 / from_nat[Real](Nat.2) <= circular_distance(Real.1 * (Real.1 / from_nat[Real](Nat.2))))
    from_nat_real_lt(Nat.2, Nat.3)
    (from_nat[Real](Nat.2) < from_nat[Real](Nat.3))
    from_nat_real_positive(Nat.2)
    (Real.0 < from_nat[Real](Nat.2))
    from_nat_real_positive(Nat.3)
    (Real.0 < from_nat[Real](Nat.3))
    inverse_on_positive_flips_inequality(from_nat[Real](Nat.2), from_nat[Real](Nat.3))
    (from_nat[Real](Nat.3).inverse < from_nat[Real](Nat.2).inverse)
    (from_nat[Real](Nat.3).inverse = Real.1 / from_nat[Real](Nat.3))
    (from_nat[Real](Nat.2).inverse = Real.1 / from_nat[Real](Nat.2))
    (Real.1 / from_nat[Real](Nat.3) < Real.1 / from_nat[Real](Nat.2))
    lt_imp_lte(Real.1 / from_nat[Real](Nat.3), Real.1 / from_nat[Real](Nat.2))
    Real.1 / from_nat[Real](Nat.3) <= Real.1 / from_nat[Real](Nat.2)
    lte_trans(Real.1 / from_nat[Real](Nat.3), Real.1 / from_nat[Real](Nat.2),
        circular_distance(Real.1 * (Real.1 / from_nat[Real](Nat.2))))
    Real.1 / from_nat[Real](Nat.3) <= circular_distance(Real.1 * (Real.1 / from_nat[Real](Nat.2)))
    exists_intro(function(t: Real) {
        Real.1 / from_nat[Real](Nat.3) <= circular_distance(Real.1 * t)
    }, Real.1 / from_nat[Real](Nat.2))
    (Real.1 / from_nat[Real](Nat.3) <= circular_distance(Real.1 * (Real.1 / from_nat[Real](Nat.2))))
    exists(t: Real) {
        Real.1 / from_nat[Real](Nat.3) <= circular_distance(Real.1 * t)
    }
}

/// The existing two-runner theorem, restated at speed one: the time `1/3`
/// works as well.  `lonely_runner_two` in analysis/real/lonely_runner.ac
/// proves the general statement for any speed `v >= 1` at time `1/(3v)`.
theorem lonely_runner_two_restated {
    exists(t: Real) {
        Real.1 / from_nat[Real](Nat.3) <= circular_distance(Real.1 * t)
    }
} by {
    lte_ref(Real.1)
    Real.1 <= Real.1
    lonely_runner_two(Real.1)
    exists(t: Real) {
        Real.1 / from_nat[Real](Nat.3) <= circular_distance(Real.1 * t)
    }
}

/// The fractional-part view of the two-runner case: at `t = 1/2` the
/// fractional part of `t` lies in `[1/3, 2/3]`.
theorem lonely_runner_two_frac_witness {
    exists(t: Real) {
        Real.1 / from_nat[Real](Nat.3) <= frac(Real.1 * t)
            and frac(Real.1 * t) <= from_nat[Real](Nat.2) / from_nat[Real](Nat.3)
    }
} by {
    lonely_runner_two_speeds_01
    exists(t: Real) {
        Real.1 / from_nat[Real](Nat.3) <= circular_distance(Real.1 * t)
    }
    let t0: Real satisfy {
        Real.1 / from_nat[Real](Nat.3) <= circular_distance(Real.1 * t0)
    }
    (Real.1 / from_nat[Real](Nat.3) <= circular_distance(Real.1 * t0))
    lonely_runner_two_frac_view(t0)
    (Real.1 / from_nat[Real](Nat.3) <= circular_distance(Real.1 * t0)
        = (Real.1 / from_nat[Real](Nat.3) <= frac(Real.1 * t0)
            and frac(Real.1 * t0) <= from_nat[Real](Nat.2) / from_nat[Real](Nat.3)))
    (Real.1 / from_nat[Real](Nat.3) <= frac(Real.1 * t0))
    (frac(Real.1 * t0) <= from_nat[Real](Nat.2) / from_nat[Real](Nat.3))
    (Real.1 / from_nat[Real](Nat.3) <= frac(Real.1 * t0)
        and frac(Real.1 * t0) <= from_nat[Real](Nat.2) / from_nat[Real](Nat.3))
    exists_intro(function(t: Real) {
        Real.1 / from_nat[Real](Nat.3) <= frac(Real.1 * t)
            and frac(Real.1 * t) <= from_nat[Real](Nat.2) / from_nat[Real](Nat.3)
    }, t0)
    (Real.1 / from_nat[Real](Nat.3) <= frac(Real.1 * t0)
        and frac(Real.1 * t0) <= from_nat[Real](Nat.2) / from_nat[Real](Nat.3))
    exists(t: Real) {
        Real.1 / from_nat[Real](Nat.3) <= frac(Real.1 * t)
            and frac(Real.1 * t) <= from_nat[Real](Nat.2) / from_nat[Real](Nat.3)
    }
}

// ----------------------------------------------------------------------------
// Section 2b: three runners, speeds 0, 1 and 2
// ----------------------------------------------------------------------------

/// Three runners (speeds 0, 1 and 2): at time 1/4 the moving runners are at
/// 1/4 and 1/2, both at circular distance at least 1/4 from the origin.  This
/// restates `lonely_runner_three` in analysis/real/lonely_runner.ac.
theorem lonely_runner_three_speeds_012 {
    exists(t: Real) {
        Real.1 / from_nat[Real](Nat.4) <= circular_distance(Real.1 * t)
            and Real.1 / from_nat[Real](Nat.4) <= circular_distance(from_nat[Real](Nat.2) * t)
    }
} by {
    mul_one_left(Real.1 / from_nat[Real](Nat.4))
    (Real.1 * (Real.1 / from_nat[Real](Nat.4)) = Real.1 / from_nat[Real](Nat.4))
    cd_bound_frac(Nat.1, Nat.4)
    (Nat.1 <= Nat.1 and Nat.1 < Nat.4)
    (Real.1 / from_nat[Real](Nat.4) <= circular_distance(from_nat[Real](Nat.1) / from_nat[Real](Nat.4)))
    (from_nat[Real](Nat.1) = Real.1)
    (Real.1 / from_nat[Real](Nat.4) <= circular_distance(Real.1 / from_nat[Real](Nat.4)))
    (Real.1 / from_nat[Real](Nat.4) <= circular_distance(Real.1 * (Real.1 / from_nat[Real](Nat.4))))
    from_nat_real_positive(Nat.4)
    (Real.0 < from_nat[Real](Nat.4))
    (from_nat[Real](Nat.4) != Real.0)
    mul_one_over(from_nat[Real](Nat.2), from_nat[Real](Nat.4))
    (from_nat[Real](Nat.2) * (Real.1 / from_nat[Real](Nat.4)) = from_nat[Real](Nat.2) / from_nat[Real](Nat.4))
    cd_bound_frac(Nat.2, Nat.4)
    (Nat.1 <= Nat.2 and Nat.2 < Nat.4)
    (Real.1 / from_nat[Real](Nat.4) <= circular_distance(from_nat[Real](Nat.2) / from_nat[Real](Nat.4)))
    (Real.1 / from_nat[Real](Nat.4) <= circular_distance(from_nat[Real](Nat.2) * (Real.1 / from_nat[Real](Nat.4))))
    (Real.1 / from_nat[Real](Nat.4)
        <= circular_distance(Real.1 * (Real.1 / from_nat[Real](Nat.4)))
        and Real.1 / from_nat[Real](Nat.4)
            <= circular_distance(from_nat[Real](Nat.2) * (Real.1 / from_nat[Real](Nat.4))))
    exists(t: Real) {
        Real.1 / from_nat[Real](Nat.4) <= circular_distance(Real.1 * t)
            and Real.1 / from_nat[Real](Nat.4) <= circular_distance(from_nat[Real](Nat.2) * t)
    }
}

/// The fractional-part view of the three-runner case: at `t = 1/4` both
/// fractional parts lie in `[1/4, 3/4]`.
theorem lonely_runner_three_frac_witness {
    exists(t: Real) {
        Real.1 / from_nat[Real](Nat.4) <= frac(Real.1 * t)
            and frac(Real.1 * t) <= from_nat[Real](Nat.3) / from_nat[Real](Nat.4)
            and Real.1 / from_nat[Real](Nat.4) <= frac(from_nat[Real](Nat.2) * t)
            and frac(from_nat[Real](Nat.2) * t) <= from_nat[Real](Nat.3) / from_nat[Real](Nat.4)
    }
} by {
    lonely_runner_three_speeds_012
    exists(t: Real) {
        Real.1 / from_nat[Real](Nat.4) <= circular_distance(Real.1 * t)
            and Real.1 / from_nat[Real](Nat.4) <= circular_distance(from_nat[Real](Nat.2) * t)
    }
    let t0: Real satisfy {
        Real.1 / from_nat[Real](Nat.4) <= circular_distance(Real.1 * t0)
            and Real.1 / from_nat[Real](Nat.4) <= circular_distance(from_nat[Real](Nat.2) * t0)
    }
    (Real.1 / from_nat[Real](Nat.4) <= circular_distance(Real.1 * t0))
    lonely_runner_three_frac_view_one(t0)
    (Real.1 / from_nat[Real](Nat.4) <= circular_distance(Real.1 * t0)
        = (Real.1 / from_nat[Real](Nat.4) <= frac(Real.1 * t0)
            and frac(Real.1 * t0) <= from_nat[Real](Nat.3) / from_nat[Real](Nat.4)))
    (Real.1 / from_nat[Real](Nat.4) <= frac(Real.1 * t0))
    (frac(Real.1 * t0) <= from_nat[Real](Nat.3) / from_nat[Real](Nat.4))
    (Real.1 / from_nat[Real](Nat.4) <= circular_distance(from_nat[Real](Nat.2) * t0))
    lonely_runner_three_frac_view_two(t0)
    (Real.1 / from_nat[Real](Nat.4) <= circular_distance(from_nat[Real](Nat.2) * t0)
        = (Real.1 / from_nat[Real](Nat.4) <= frac(from_nat[Real](Nat.2) * t0)
            and frac(from_nat[Real](Nat.2) * t0) <= from_nat[Real](Nat.3) / from_nat[Real](Nat.4)))
    (Real.1 / from_nat[Real](Nat.4) <= frac(from_nat[Real](Nat.2) * t0))
    (frac(from_nat[Real](Nat.2) * t0) <= from_nat[Real](Nat.3) / from_nat[Real](Nat.4))
    (Real.1 / from_nat[Real](Nat.4) <= frac(Real.1 * t0)
        and frac(Real.1 * t0) <= from_nat[Real](Nat.3) / from_nat[Real](Nat.4)
        and Real.1 / from_nat[Real](Nat.4) <= frac(from_nat[Real](Nat.2) * t0)
        and frac(from_nat[Real](Nat.2) * t0) <= from_nat[Real](Nat.3) / from_nat[Real](Nat.4))
    exists_intro(function(t: Real) {
        Real.1 / from_nat[Real](Nat.4) <= frac(Real.1 * t)
            and frac(Real.1 * t) <= from_nat[Real](Nat.3) / from_nat[Real](Nat.4)
            and Real.1 / from_nat[Real](Nat.4) <= frac(from_nat[Real](Nat.2) * t)
            and frac(from_nat[Real](Nat.2) * t) <= from_nat[Real](Nat.3) / from_nat[Real](Nat.4)
    }, t0)
    (Real.1 / from_nat[Real](Nat.4) <= frac(Real.1 * t0)
        and frac(Real.1 * t0) <= from_nat[Real](Nat.3) / from_nat[Real](Nat.4)
        and Real.1 / from_nat[Real](Nat.4) <= frac(from_nat[Real](Nat.2) * t0)
        and frac(from_nat[Real](Nat.2) * t0) <= from_nat[Real](Nat.3) / from_nat[Real](Nat.4))
    exists(t: Real) {
        Real.1 / from_nat[Real](Nat.4) <= frac(Real.1 * t)
            and frac(Real.1 * t) <= from_nat[Real](Nat.3) / from_nat[Real](Nat.4)
            and Real.1 / from_nat[Real](Nat.4) <= frac(from_nat[Real](Nat.2) * t)
            and frac(from_nat[Real](Nat.2) * t) <= from_nat[Real](Nat.3) / from_nat[Real](Nat.4)
    }
}

// ============================================================================
// Section 3: the equispaced case, speeds 0, 1, ..., n
// ============================================================================

/// The equispaced case: runners with speeds `0, 1, 2, ..., n` (that is,
/// `n + 1` runners in total) are all lonely at the single time
/// `t = 1 / (n + 2)`.
///
/// At that time runner `i` sits at the lattice point `i / (n + 2)`, whose
/// circular distance from the origin is `min(i, n + 2 - i) / (n + 2)`, at
/// least `1 / (n + 2)` for every `1 <= i <= n`.  The threshold `1 / (n + 2)`
/// is the lonely bound for `n + 1` runners.  This generalizes the concrete
/// witnesses `t = 1/4`, `t = 1/5` and `t = 1/6` of the small cases.  The half
/// time `t = 1 / (2 (n + 1))` does not work: the runners are then spread over
/// `[0, 1/2)` with adjacent gap exactly `1 / (2 (n + 1))`, below `1 / (n + 1)`.
theorem lonely_runner_equispaced(n: Nat) {
    exists(t: Real) {
        forall(i: Nat) {
            Nat.1 <= i and i <= n implies
                Real.1 / from_nat[Real](n + Nat.2) <= circular_distance(from_nat[Real](i) * t)
        }
    }
} by {
    lt_add_suc(Nat.0, n + Nat.1)
    (Nat.0 < Nat.0 + (n + Nat.1).suc)
    (Nat.0 + (n + Nat.1).suc = n + Nat.2)
    Nat.0 < n + Nat.2
    from_nat_real_positive(n + Nat.2)
    (Real.0 < from_nat[Real](n + Nat.2))
    exists_intro(function(t: Real) {
        forall(i: Nat) {
            Nat.1 <= i and i <= n implies
                Real.1 / from_nat[Real](n + Nat.2) <= circular_distance(from_nat[Real](i) * t)
        }
    }, Real.1 / from_nat[Real](n + Nat.2))
    forall(i: Nat) {
        if Nat.1 <= i and i <= n {
            lt_add_suc(n, Nat.1)
            (n < n + Nat.1.suc)
            (n + Nat.1.suc = n + Nat.2)
            n < n + Nat.2
            lte_and_lt(i, n, n + Nat.2)
            (i <= n and n < n + Nat.2 implies i < n + Nat.2)
            i < n + Nat.2
            cd_bound_frac(i, n + Nat.2)
            (Nat.1 <= i and i < n + Nat.2)
            (Real.1 / from_nat[Real](n + Nat.2) <= circular_distance(
                from_nat[Real](i) / from_nat[Real](n + Nat.2)))
            from_nat_real_positive(n + Nat.2)
            (Real.0 < from_nat[Real](n + Nat.2))
            (from_nat[Real](n + Nat.2) != Real.0)
            mul_one_over(from_nat[Real](i), from_nat[Real](n + Nat.2))
            (from_nat[Real](i) * (Real.1 / from_nat[Real](n + Nat.2))
                = from_nat[Real](i) / from_nat[Real](n + Nat.2))
            (Real.1 / from_nat[Real](n + Nat.2) <= circular_distance(
                from_nat[Real](i) * (Real.1 / from_nat[Real](n + Nat.2))))
        }
    }
}

/// The equispaced case in the fractional-part view: at `t = 1 / (n + 2)` every
/// moving runner `i` has `frac(i * t) = i / (n + 2)` lying in
/// `[1 / (n + 2), (n + 1) / (n + 2)]`.
theorem lonely_runner_equispaced_frac_view(n: Nat) {
    exists(t: Real) {
        forall(i: Nat) {
            Nat.1 <= i and i <= n implies
                Real.1 / from_nat[Real](n + Nat.2) <= frac(from_nat[Real](i) * t)
                and frac(from_nat[Real](i) * t)
                    <= from_nat[Real](n + Nat.1) / from_nat[Real](n + Nat.2)
        }
    }
} by {
    lonely_runner_equispaced(n)
    exists(t: Real) {
        forall(i: Nat) {
            Nat.1 <= i and i <= n implies
                Real.1 / from_nat[Real](n + Nat.2) <= circular_distance(from_nat[Real](i) * t)
        }
    }
    let t0: Real satisfy {
        forall(i: Nat) {
            Nat.1 <= i and i <= n implies
                Real.1 / from_nat[Real](n + Nat.2) <= circular_distance(from_nat[Real](i) * t0)
        }
    }
    exists_intro(function(t: Real) {
        forall(i: Nat) {
            Nat.1 <= i and i <= n implies
                Real.1 / from_nat[Real](n + Nat.2) <= frac(from_nat[Real](i) * t)
                and frac(from_nat[Real](i) * t)
                    <= from_nat[Real](n + Nat.1) / from_nat[Real](n + Nat.2)
        }
    }, t0)
    forall(i: Nat) {
        if Nat.1 <= i and i <= n {
            (Nat.1 <= i and i <= n implies
                Real.1 / from_nat[Real](n + Nat.2) <= circular_distance(from_nat[Real](i) * t0))
            (Real.1 / from_nat[Real](n + Nat.2) <= circular_distance(from_nat[Real](i) * t0))
            cd_iff_frac_interval(from_nat[Real](i) * t0, Real.1 / from_nat[Real](n + Nat.2))
            (Real.1 / from_nat[Real](n + Nat.2) <= circular_distance(from_nat[Real](i) * t0)
                = (Real.1 / from_nat[Real](n + Nat.2) <= frac(from_nat[Real](i) * t0)
                    and frac(from_nat[Real](i) * t0)
                        <= Real.1 - Real.1 / from_nat[Real](n + Nat.2)))
            (Real.1 / from_nat[Real](n + Nat.2) <= frac(from_nat[Real](i) * t0)
                and frac(from_nat[Real](i) * t0)
                    <= Real.1 - Real.1 / from_nat[Real](n + Nat.2))
            nat_one_lt_add_two(n)
            (Nat.1 < n + Nat.2)
            one_minus_frac(Nat.1, n + Nat.2)
            (Real.1 - from_nat[Real](Nat.1) / from_nat[Real](n + Nat.2)
                = from_nat[Real](n + Nat.2 - Nat.1) / from_nat[Real](n + Nat.2))
            (from_nat[Real](Nat.1) = Real.1)
            (Real.1 - Real.1 / from_nat[Real](n + Nat.2)
                = from_nat[Real](n + Nat.2 - Nat.1) / from_nat[Real](n + Nat.2))
            nat_add_two_sub_one(n)
            (n + Nat.2 - Nat.1 = n + Nat.1)
            (from_nat[Real](n + Nat.2 - Nat.1) = from_nat[Real](n + Nat.1))
            (Real.1 - Real.1 / from_nat[Real](n + Nat.2)
                = from_nat[Real](n + Nat.1) / from_nat[Real](n + Nat.2))
            (Real.1 / from_nat[Real](n + Nat.2) <= frac(from_nat[Real](i) * t0)
                and frac(from_nat[Real](i) * t0)
                    <= from_nat[Real](n + Nat.1) / from_nat[Real](n + Nat.2))
        }
    }
    exists(t: Real) {
        forall(i: Nat) {
            Nat.1 <= i and i <= n implies
                Real.1 / from_nat[Real](n + Nat.2) <= frac(from_nat[Real](i) * t)
                and frac(from_nat[Real](i) * t)
                    <= from_nat[Real](n + Nat.1) / from_nat[Real](n + Nat.2)
        }
    }
}

// ============================================================================
// Section 4: the full conjecture (open problem)
// ============================================================================
//
// The lonely runner conjecture: for any number m of runners, with runner 0
// stationary at the origin and runners 1..m-1 moving at distinct positive
// integer speeds, there is a time t at which every moving runner is at
// circular distance at least 1/(m+1) from the origin.  Equivalently, in the
// lonely-runner view of Section 1: for some t, the fractional part of v_i * t
// lies in [1/(m+1), m/(m+1)] for every moving runner i (by
// `cd_iff_frac_interval`).  The conjecture is known for up to seven moving
// runners and is open in general; it is therefore stated here but not proved.
//
// theorem lonely_runner_conjecture(m: Nat, v: Nat -> Nat) {
//     v(Nat.0) = Nat.0
//         and forall(i: Nat) {
//             Nat.1 <= i and i < m implies v(i) < v(i + Nat.1)
//         }
//     implies exists(t: Real) {
//         forall(i: Nat) {
//             Nat.1 <= i and i < m implies
//                 Real.1 / from_nat[Real](m + Nat.1) <= circular_distance(from_nat[Real](v(i)) * t)
//         }
//     }
// } by {
//     // Open problem; not proved in this file.
// }
