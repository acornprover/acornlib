from int import Int, exp_add, exp_mul, exp_one, one_exp, sq_eq_mul, mul_neg_left,
    mul_one_left, neg_neg
from nat import Nat, div_mod_decomp, mod_lt, div_mul, lt_suc, lte_trans, lt_imp_lte_suc,
    pos_of_ne_zero, lte_mul_both, lt_or_lte, not_lt_zero, distrib_left, distrib_right,
    mul_assoc, mul_comm, add_cancels_left, add_imp_sub_left, small_mod, two_divides_suc_iff,
    lt_suc_right, sub_lt, lt_trans
from number_theory.legendre_symbol import legendre_symbol
from number_theory.quadratic_residue_supplements import prime_pred_legendre_symbol_by_half_parity,
    prime_two_legendre_symbol_mod_eight, double_add_one_mod_eight_from_mod_four,
    odd_decomp_of_not_two_divides, prime_two_quadratic_residue_iff_congr_one_or_seven_mod_eight
from number_theory.primitive_root import is_order_double_unit_generator_mod
from number_theory.fermat import prime_divides_mul
from number_theory.carmichael import two_is_prime
numerals Int

// ============================================================================
// Quadratic reciprocity applications: the supplementary laws in closed form
// ============================================================================
//
// The law of quadratic reciprocity itself is not yet proved in this library;
// the Gauss-lemma machinery of quadratic_reciprocity.ac and the two
// supplementary laws proved in quadratic_residue_supplements.ac are.  This
// file restates the supplementary laws with the classical power-of-minus-one
// formulas:
//
//     (-1/p) = (-1)^((p-1)/2)      and      (2/p) = (-1)^((p²-1)/8),
//
// which is how the supplements are usually applied to compute Legendre
// symbols.  Writing p = 2h + 1, the first formula is (-1)^h and the second
// is (-1)^(h(h+1)/2), since (p²-1)/8 = ((2h+1)²-1)/8 = h(h+1)/2.
//
// The full reciprocity law, for odd primes p and q,
//
//     (p/q)(q/p) = (-1)^(((p-1)/2)((q-1)/2)),
//
// needs the Gauss count of q modulo p to be even exactly when
// ((p-1)/2)((q-1)/2) is even — the counting argument over the rectangle of
// lattice points below the line px - qy = 0 (Eisenstein's proof) — which is
// not yet formalised.  It is recorded as a statement:
//
// theorem quadratic_reciprocity_law(p: Nat, q: Nat, hp: Nat, hq: Nat) {
//     p.is_prime and q.is_prime and p != Nat.2 and q != Nat.2 and
//         p = Nat.2 * hp + Nat.1 and q = Nat.2 * hq + Nat.1
//     implies legendre_symbol(p, q) * legendre_symbol(q, p) =
//         (-Int.1).pow(hp * hq)
// }

/// Minus one to an even power is one.
theorem int_neg_one_pow_even(k: Nat) {
    (-Int.1).pow(Nat.2 * k) = Int.1
} by {
    exp_mul(-Int.1, Nat.2, k)
    (-Int.1).pow(Nat.2 * k) = (-Int.1).pow(Nat.2).pow(k)
    sq_eq_mul(-Int.1)
    (-Int.1) * (-Int.1) = (-Int.1).pow(Nat.2)
    mul_neg_left(Int.1, -Int.1)
    (-Int.1) * (-Int.1) = -(Int.1 * (-Int.1))
    mul_one_left(-Int.1)
    Int.1 * (-Int.1) = -Int.1
    -(Int.1 * (-Int.1)) = -(-Int.1)
    neg_neg(Int.1)
    -(-Int.1) = Int.1
    (-Int.1) * (-Int.1) = Int.1
    (-Int.1).pow(Nat.2) = Int.1
    (-Int.1).pow(Nat.2).pow(k) = Int.1.pow(k)
    one_exp(k)
    Int.1.pow(k) = Int.1
    (-Int.1).pow(Nat.2 * k) = Int.1
}

/// Minus one to an odd power is minus one.
theorem int_neg_one_pow_odd(k: Nat) {
    (-Int.1).pow(Nat.2 * k + Nat.1) = -Int.1
} by {
    exp_add(-Int.1, Nat.2 * k, Nat.1)
    (-Int.1).pow(Nat.2 * k + Nat.1) =
        (-Int.1).pow(Nat.2 * k) * (-Int.1).pow(Nat.1)
    int_neg_one_pow_even(k)
    (-Int.1).pow(Nat.2 * k) = Int.1
    exp_one(-Int.1)
    (-Int.1).pow(Nat.1) = -Int.1
    Int.1 * (-Int.1) = -Int.1
    (-Int.1).pow(Nat.2 * k + Nat.1) = -Int.1
}

/// Minus one to the power of a natural is one or minus one according to its
/// parity.
theorem int_neg_one_pow_parity(h: Nat) {
    (-Int.1).pow(h) = if Nat.2.divides(h) { Int.1 } else { -Int.1 }
} by {
    if Nat.2.divides(h) {
        Nat.2.divides(h) = exists(c: Nat) { Nat.2 * c = h }
        let (c: Nat) satisfy { Nat.2 * c = h }
        int_neg_one_pow_even(c)
        (-Int.1).pow(Nat.2 * c) = Int.1
        h = Nat.2 * c
        (-Int.1).pow(h) = Int.1
        (-Int.1).pow(h) = if Nat.2.divides(h) { Int.1 } else { -Int.1 }
    } else {
        not Nat.2.divides(h)
        odd_decomp_of_not_two_divides(h)
        exists(q: Nat) { h = Nat.2 * q + Nat.1 }
        let (c: Nat) satisfy { h = Nat.2 * c + Nat.1 }
        int_neg_one_pow_odd(c)
        (-Int.1).pow(Nat.2 * c + Nat.1) = -Int.1
        h = Nat.2 * c + Nat.1
        (-Int.1).pow(h) = -Int.1
        (-Int.1).pow(h) = if Nat.2.divides(h) { Int.1 } else { -Int.1 }
    }
    (-Int.1).pow(h) = if Nat.2.divides(h) { Int.1 } else { -Int.1 }
}

/// A number strictly below four is zero, one, two, or three.
theorem nat_lt_four_cases_local(r: Nat) {
    r < Nat.4 implies r = Nat.0 or r = Nat.1 or r = Nat.2 or r = Nat.3
} by {
    if r < Nat.4 {
        lt_suc_right(r, Nat.3)
        if r = Nat.3 {
            r = Nat.3
            r = Nat.0 or r = Nat.1 or r = Nat.2 or r = Nat.3
        } else {
            r < Nat.3
            lt_suc_right(r, Nat.2)
            if r = Nat.2 {
                r = Nat.2
                r = Nat.0 or r = Nat.1 or r = Nat.2 or r = Nat.3
            } else {
                r < Nat.2
                lt_suc_right(r, Nat.1)
                if r = Nat.1 {
                    r = Nat.1
                    r = Nat.0 or r = Nat.1 or r = Nat.2 or r = Nat.3
                } else {
                    r < Nat.1
                    lt_suc_right(r, Nat.0)
                    not_lt_zero(r)
                    r = Nat.0
                    r = Nat.0 or r = Nat.1 or r = Nat.2 or r = Nat.3
                }
            }
        }
        r = Nat.0 or r = Nat.1 or r = Nat.2 or r = Nat.3
    }
}

/// Twice a natural number plus one is not divisible by two.
theorem nat_two_not_divides_double_add_one(q: Nat) {
    not Nat.2.divides(Nat.2 * q + Nat.1)
} by {
    Nat.2 * q + Nat.1 = (Nat.2 * q).suc
    two_divides_suc_iff(Nat.2 * q)
    Nat.2.divides((Nat.2 * q).suc) = not Nat.2.divides(Nat.2 * q)
    Nat.2 * q = Nat.2 * q
    exists(c: Nat) { Nat.2 * c = Nat.2 * q }
    Nat.2.divides(Nat.2 * q)
    not Nat.2.divides((Nat.2 * q).suc)
    Nat.2 * q + Nat.1 = (Nat.2 * q).suc
    not Nat.2.divides(Nat.2 * q + Nat.1)
}

/// The first supplementary law in closed form: (-1/p) = (-1)^((p-1)/2).
theorem prime_neg_one_legendre_symbol_pow(p: Nat, h: Nat) {
    p.is_prime and p = Nat.2 * h + Nat.1 implies
        legendre_symbol(p - Nat.1, p) = (-Int.1).pow(h)
} by {
    if p.is_prime and p = Nat.2 * h + Nat.1 {
        prime_pred_legendre_symbol_by_half_parity(p, h)
        legendre_symbol(p - Nat.1, p) =
            if Nat.2.divides(h) { Int.1 } else { -Int.1 }
        int_neg_one_pow_parity(h)
        (-Int.1).pow(h) = if Nat.2.divides(h) { Int.1 } else { -Int.1 }
        legendre_symbol(p - Nat.1, p) = (-Int.1).pow(h)
    }
}

/// Twice the remainder of h modulo four plus one is the remainder of 2h+1
/// modulo eight.
theorem double_add_one_mod_eight_from_mod_four_form(h: Nat) {
    (Nat.2 * h + Nat.1).mod(Nat.8) = Nat.2 * h.mod(Nat.4) + Nat.1
} by {
    double_add_one_mod_eight_from_mod_four(h)
    (Nat.2 * h + Nat.1).mod(Nat.8) = Nat.2 * h.mod(Nat.4) + Nat.1
}

/// A number of the form 2h+1 is congruent to one or seven modulo eight
/// exactly when h is congruent to zero or three modulo four.
theorem double_add_one_mod_eight_cases(h: Nat) {
    ((Nat.2 * h + Nat.1).mod(Nat.8) = Nat.1 or
        (Nat.2 * h + Nat.1).mod(Nat.8) = Nat.7) =
        (h.mod(Nat.4) = Nat.0 or h.mod(Nat.4) = Nat.3)
} by {
    div_mod_decomp(h, Nat.4)
    h.div(Nat.4) * Nat.4 + h.mod(Nat.4) = h
    mod_lt(h, Nat.4)
    h.mod(Nat.4) < Nat.4
    nat_lt_four_cases_local(h.mod(Nat.4))
    if h.mod(Nat.4) = Nat.0 {
        double_add_one_mod_eight_from_mod_four_form(h)
        (Nat.2 * h + Nat.1).mod(Nat.8) = Nat.2 * Nat.0 + Nat.1
        Nat.2 * Nat.0 = Nat.0
        (Nat.2 * h + Nat.1).mod(Nat.8) = Nat.1
        h.mod(Nat.4) = Nat.0
        ((Nat.2 * h + Nat.1).mod(Nat.8) = Nat.1 or
            (Nat.2 * h + Nat.1).mod(Nat.8) = Nat.7) =
            (h.mod(Nat.4) = Nat.0 or h.mod(Nat.4) = Nat.3)
    } else {
        if h.mod(Nat.4) = Nat.1 {
            double_add_one_mod_eight_from_mod_four_form(h)
            (Nat.2 * h + Nat.1).mod(Nat.8) = Nat.2 * Nat.1 + Nat.1
            Nat.2 * Nat.1 = Nat.2
            (Nat.2 * h + Nat.1).mod(Nat.8) = Nat.3
            if (Nat.2 * h + Nat.1).mod(Nat.8) = Nat.1 or
                (Nat.2 * h + Nat.1).mod(Nat.8) = Nat.7 {
                (Nat.2 * h + Nat.1).mod(Nat.8) = Nat.1
                Nat.3 = Nat.1
                false
            }
            if h.mod(Nat.4) = Nat.0 or h.mod(Nat.4) = Nat.3 {
                h.mod(Nat.4) = Nat.0
                Nat.1 = Nat.0
                false
            }
            ((Nat.2 * h + Nat.1).mod(Nat.8) = Nat.1 or
                (Nat.2 * h + Nat.1).mod(Nat.8) = Nat.7) =
                (h.mod(Nat.4) = Nat.0 or h.mod(Nat.4) = Nat.3)
        } else {
            if h.mod(Nat.4) = Nat.2 {
                double_add_one_mod_eight_from_mod_four_form(h)
                (Nat.2 * h + Nat.1).mod(Nat.8) = Nat.2 * Nat.2 + Nat.1
                Nat.2 * Nat.2 = Nat.4
                (Nat.2 * h + Nat.1).mod(Nat.8) = Nat.5
                if (Nat.2 * h + Nat.1).mod(Nat.8) = Nat.1 or
                    (Nat.2 * h + Nat.1).mod(Nat.8) = Nat.7 {
                    (Nat.2 * h + Nat.1).mod(Nat.8) = Nat.1
                    Nat.5 = Nat.1
                    false
                }
                if h.mod(Nat.4) = Nat.0 or h.mod(Nat.4) = Nat.3 {
                    h.mod(Nat.4) = Nat.0
                    Nat.2 = Nat.0
                    false
                }
                ((Nat.2 * h + Nat.1).mod(Nat.8) = Nat.1 or
                    (Nat.2 * h + Nat.1).mod(Nat.8) = Nat.7) =
                    (h.mod(Nat.4) = Nat.0 or h.mod(Nat.4) = Nat.3)
            } else {
                h.mod(Nat.4) = Nat.3
                double_add_one_mod_eight_from_mod_four_form(h)
                (Nat.2 * h + Nat.1).mod(Nat.8) = Nat.2 * Nat.3 + Nat.1
                Nat.2 * Nat.3 = Nat.6
                (Nat.2 * h + Nat.1).mod(Nat.8) = Nat.7
                h.mod(Nat.4) = Nat.3
                ((Nat.2 * h + Nat.1).mod(Nat.8) = Nat.1 or
                    (Nat.2 * h + Nat.1).mod(Nat.8) = Nat.7) =
                    (h.mod(Nat.4) = Nat.0 or h.mod(Nat.4) = Nat.3)
            }
        }
    }
    ((Nat.2 * h + Nat.1).mod(Nat.8) = Nat.1 or
        (Nat.2 * h + Nat.1).mod(Nat.8) = Nat.7) =
        (h.mod(Nat.4) = Nat.0 or h.mod(Nat.4) = Nat.3)
}

/// The halved product h(h+1)/2 is even when h is congruent to zero or three
/// modulo four.
theorem nat_half_prod_div_two_even_of_mod_four_zero_or_three(h: Nat) {
    (h.mod(Nat.4) = Nat.0 or h.mod(Nat.4) = Nat.3) implies
        Nat.2.divides((h * (h + Nat.1)).div(Nat.2))
} by {
    if h.mod(Nat.4) = Nat.0 or h.mod(Nat.4) = Nat.3 {
        if h.mod(Nat.4) = Nat.0 {
            div_mod_decomp(h, Nat.4)
            let (q: Nat) satisfy { q * Nat.4 + h.mod(Nat.4) = h }
            h = q * Nat.4 + Nat.0
            h = q * Nat.4
            Nat.4 = Nat.2 * Nat.2
            h = q * (Nat.2 * Nat.2)
            mul_assoc(q, Nat.2, Nat.2)
            (q * Nat.2) * Nat.2 = q * (Nat.2 * Nat.2)
            h = (q * Nat.2) * Nat.2
            h = Nat.2 * (q * Nat.2)
            h * (h + Nat.1) = (Nat.2 * (q * Nat.2)) * (h + Nat.1)
            Nat.2 * (q * Nat.2) * (h + Nat.1) = (q * Nat.2) * Nat.2 * (h + Nat.1)
            (q * Nat.2) * Nat.2 * (h + Nat.1) = ((q * Nat.2) * (h + Nat.1)) * Nat.2
            h * (h + Nat.1) = ((q * Nat.2) * (h + Nat.1)) * Nat.2
            (h * (h + Nat.1)).div(Nat.2) =
                (((q * Nat.2) * (h + Nat.1)) * Nat.2).div(Nat.2)
            div_mul((q * Nat.2) * (h + Nat.1), Nat.2)
            (((q * Nat.2) * (h + Nat.1)) * Nat.2).div(Nat.2) =
                (q * Nat.2) * (h + Nat.1)
            (h * (h + Nat.1)).div(Nat.2) = (q * Nat.2) * (h + Nat.1)
            (q * Nat.2) * (h + Nat.1) = Nat.2 * (q * (h + Nat.1))
            (h * (h + Nat.1)).div(Nat.2) = Nat.2 * (q * (h + Nat.1))
            exists(c: Nat) { Nat.2 * c = (h * (h + Nat.1)).div(Nat.2) }
            Nat.2.divides((h * (h + Nat.1)).div(Nat.2))
        } else {
            h.mod(Nat.4) = Nat.3
            div_mod_decomp(h, Nat.4)
            let (q: Nat) satisfy { q * Nat.4 + h.mod(Nat.4) = h }
            h = q * Nat.4 + Nat.3
            h + Nat.1 = q * Nat.4 + Nat.4
            q * Nat.4 + Nat.4 = (q + Nat.1) * Nat.4
            h + Nat.1 = (q + Nat.1) * Nat.4
            Nat.4 = Nat.2 * Nat.2
            h + Nat.1 = (q + Nat.1) * (Nat.2 * Nat.2)
            mul_assoc(q + Nat.1, Nat.2, Nat.2)
            ((q + Nat.1) * Nat.2) * Nat.2 = (q + Nat.1) * (Nat.2 * Nat.2)
            h + Nat.1 = ((q + Nat.1) * Nat.2) * Nat.2
            h + Nat.1 = Nat.2 * ((q + Nat.1) * Nat.2)
            h * (h + Nat.1) = h * (Nat.2 * ((q + Nat.1) * Nat.2))
            h * (Nat.2 * ((q + Nat.1) * Nat.2)) =
                (h * ((q + Nat.1) * Nat.2)) * Nat.2
            h * (h + Nat.1) = (h * ((q + Nat.1) * Nat.2)) * Nat.2
            (h * (h + Nat.1)).div(Nat.2) =
                ((h * ((q + Nat.1) * Nat.2)) * Nat.2).div(Nat.2)
            div_mul(h * ((q + Nat.1) * Nat.2), Nat.2)
            ((h * ((q + Nat.1) * Nat.2)) * Nat.2).div(Nat.2) =
                h * ((q + Nat.1) * Nat.2)
            (h * (h + Nat.1)).div(Nat.2) = h * ((q + Nat.1) * Nat.2)
            h * ((q + Nat.1) * Nat.2) = Nat.2 * (h * (q + Nat.1))
            (h * (h + Nat.1)).div(Nat.2) = Nat.2 * (h * (q + Nat.1))
            exists(c: Nat) { Nat.2 * c = (h * (h + Nat.1)).div(Nat.2) }
            Nat.2.divides((h * (h + Nat.1)).div(Nat.2))
        }
        Nat.2.divides((h * (h + Nat.1)).div(Nat.2))
    }
}

/// The halved product h(h+1)/2 is odd when h is congruent to one or two
/// modulo four.
theorem nat_half_prod_div_two_odd_of_mod_four_one_or_two(h: Nat) {
    (h.mod(Nat.4) = Nat.1 or h.mod(Nat.4) = Nat.2) implies
        not Nat.2.divides((h * (h + Nat.1)).div(Nat.2))
} by {
    if h.mod(Nat.4) = Nat.1 or h.mod(Nat.4) = Nat.2 {
        if h.mod(Nat.4) = Nat.1 {
            div_mod_decomp(h, Nat.4)
            let (q: Nat) satisfy { q * Nat.4 + h.mod(Nat.4) = h }
            h = q * Nat.4 + Nat.1
            Nat.4 = Nat.2 * Nat.2
            h = q * (Nat.2 * Nat.2) + Nat.1
            mul_assoc(q, Nat.2, Nat.2)
            (q * Nat.2) * Nat.2 = q * (Nat.2 * Nat.2)
            h = (q * Nat.2) * Nat.2 + Nat.1
            h = Nat.2 * (q * Nat.2) + Nat.1
            h + Nat.1 = Nat.2 * (q * Nat.2) + Nat.2
            h + Nat.1 = Nat.2 * (q * Nat.2 + Nat.1)
            h * (h + Nat.1) = (Nat.2 * (q * Nat.2) + Nat.1) * (Nat.2 * (q * Nat.2 + Nat.1))
            (Nat.2 * (q * Nat.2) + Nat.1) * (Nat.2 * (q * Nat.2 + Nat.1)) =
                ((Nat.2 * (q * Nat.2) + Nat.1) * (q * Nat.2 + Nat.1)) * Nat.2
            h * (h + Nat.1) = ((Nat.2 * (q * Nat.2) + Nat.1) * (q * Nat.2 + Nat.1)) * Nat.2
            (h * (h + Nat.1)).div(Nat.2) =
                (((Nat.2 * (q * Nat.2) + Nat.1) * (q * Nat.2 + Nat.1)) * Nat.2).div(Nat.2)
            Nat.2 != Nat.0
            div_mul((Nat.2 * (q * Nat.2) + Nat.1) * (q * Nat.2 + Nat.1), Nat.2)
            (((Nat.2 * (q * Nat.2) + Nat.1) * (q * Nat.2 + Nat.1)) * Nat.2).div(Nat.2) =
                (Nat.2 * (q * Nat.2) + Nat.1) * (q * Nat.2 + Nat.1)
            (h * (h + Nat.1)).div(Nat.2) =
                (Nat.2 * (q * Nat.2) + Nat.1) * (q * Nat.2 + Nat.1)
            (Nat.2 * (q * Nat.2) + Nat.1) * (q * Nat.2 + Nat.1) =
                Nat.2 * ((q * Nat.2) * (q * Nat.2 + Nat.1)) + (q * Nat.2 + Nat.1)
            if Nat.2.divides((h * (h + Nat.1)).div(Nat.2)) {
                two_is_prime
                Nat.2.is_prime
                prime_divides_mul(Nat.2, Nat.2 * (q * Nat.2) + Nat.1, q * Nat.2 + Nat.1)
                Nat.2.divides(Nat.2 * (q * Nat.2) + Nat.1) or
                    Nat.2.divides(q * Nat.2 + Nat.1)
                if Nat.2.divides(Nat.2 * (q * Nat.2) + Nat.1) {
                    nat_two_not_divides_double_add_one(q * Nat.2)
                    not Nat.2.divides(Nat.2 * (q * Nat.2) + Nat.1)
                    false
                } else {
                    Nat.2.divides(q * Nat.2 + Nat.1)
                    nat_two_not_divides_double_add_one(q)
                    not Nat.2.divides(Nat.2 * q + Nat.1)
                    Nat.2 * q = q * Nat.2
                    Nat.2 * q + Nat.1 = q * Nat.2 + Nat.1
                    false
                }
                false
            }
            not Nat.2.divides((h * (h + Nat.1)).div(Nat.2))
        } else {
            h.mod(Nat.4) = Nat.2
            div_mod_decomp(h, Nat.4)
            let (q: Nat) satisfy { q * Nat.4 + h.mod(Nat.4) = h }
            h = q * Nat.4 + Nat.2
            Nat.4 = Nat.2 * Nat.2
            h = q * (Nat.2 * Nat.2) + Nat.2
            mul_assoc(q, Nat.2, Nat.2)
            (q * Nat.2) * Nat.2 = q * (Nat.2 * Nat.2)
            h = (q * Nat.2) * Nat.2 + Nat.2
            h = Nat.2 * (q * Nat.2 + Nat.1)
            h + Nat.1 = Nat.2 * (q * Nat.2 + Nat.1) + Nat.1
            h * (h + Nat.1) = (Nat.2 * (q * Nat.2 + Nat.1)) * (Nat.2 * (q * Nat.2 + Nat.1) + Nat.1)
            (Nat.2 * (q * Nat.2 + Nat.1)) * (Nat.2 * (q * Nat.2 + Nat.1) + Nat.1) =
                ((q * Nat.2 + Nat.1) * (Nat.2 * (q * Nat.2 + Nat.1) + Nat.1)) * Nat.2
            h * (h + Nat.1) = ((q * Nat.2 + Nat.1) * (Nat.2 * (q * Nat.2 + Nat.1) + Nat.1)) * Nat.2
            (h * (h + Nat.1)).div(Nat.2) =
                (((q * Nat.2 + Nat.1) * (Nat.2 * (q * Nat.2 + Nat.1) + Nat.1)) * Nat.2).div(Nat.2)
            Nat.2 != Nat.0
            div_mul((q * Nat.2 + Nat.1) * (Nat.2 * (q * Nat.2 + Nat.1) + Nat.1), Nat.2)
            (((q * Nat.2 + Nat.1) * (Nat.2 * (q * Nat.2 + Nat.1) + Nat.1)) * Nat.2).div(Nat.2) =
                (q * Nat.2 + Nat.1) * (Nat.2 * (q * Nat.2 + Nat.1) + Nat.1)
            (h * (h + Nat.1)).div(Nat.2) =
                (q * Nat.2 + Nat.1) * (Nat.2 * (q * Nat.2 + Nat.1) + Nat.1)
            (q * Nat.2 + Nat.1) * (Nat.2 * (q * Nat.2 + Nat.1) + Nat.1) =
                (q * Nat.2 + Nat.1) * (Nat.2 * (q * Nat.2 + Nat.1)) + (q * Nat.2 + Nat.1) * Nat.1
            (q * Nat.2 + Nat.1) * (Nat.2 * (q * Nat.2 + Nat.1)) =
                Nat.2 * ((q * Nat.2 + Nat.1) * (q * Nat.2 + Nat.1))
            (q * Nat.2 + Nat.1) * Nat.1 = q * Nat.2 + Nat.1
            (q * Nat.2 + Nat.1) * (Nat.2 * (q * Nat.2 + Nat.1) + Nat.1) =
                Nat.2 * ((q * Nat.2 + Nat.1) * (q * Nat.2 + Nat.1)) + (q * Nat.2 + Nat.1)
            if Nat.2.divides((h * (h + Nat.1)).div(Nat.2)) {
                two_is_prime
                Nat.2.is_prime
                prime_divides_mul(Nat.2, q * Nat.2 + Nat.1,
                    Nat.2 * (q * Nat.2 + Nat.1) + Nat.1)
                Nat.2.divides(q * Nat.2 + Nat.1) or
                    Nat.2.divides(Nat.2 * (q * Nat.2 + Nat.1) + Nat.1)
                if Nat.2.divides(q * Nat.2 + Nat.1) {
                    nat_two_not_divides_double_add_one(q)
                    not Nat.2.divides(Nat.2 * q + Nat.1)
                    Nat.2 * q = q * Nat.2
                    Nat.2 * q + Nat.1 = q * Nat.2 + Nat.1
                    false
                } else {
                    Nat.2.divides(Nat.2 * (q * Nat.2 + Nat.1) + Nat.1)
                    nat_two_not_divides_double_add_one(q * Nat.2)
                    not Nat.2.divides(Nat.2 * (q * Nat.2) + Nat.1)
                    Nat.2 * (q * Nat.2) + Nat.1 = Nat.2 * (q * Nat.2 + Nat.1) + Nat.1
                    false
                }
                false
            }
            not Nat.2.divides((h * (h + Nat.1)).div(Nat.2))
        }
        not Nat.2.divides((h * (h + Nat.1)).div(Nat.2))
    }
}

/// The parity of h(h+1)/2 is governed by the residue of h modulo four.
theorem nat_half_prod_div_two_parity(h: Nat) {
    (Nat.2.divides((h * (h + Nat.1)).div(Nat.2)) =
        (h.mod(Nat.4) = Nat.0 or h.mod(Nat.4) = Nat.3))
} by {
    if Nat.2.divides((h * (h + Nat.1)).div(Nat.2)) {
        if h.mod(Nat.4) = Nat.1 or h.mod(Nat.4) = Nat.2 {
            nat_half_prod_div_two_odd_of_mod_four_one_or_two(h)
            not Nat.2.divides((h * (h + Nat.1)).div(Nat.2))
            false
        }
        not (h.mod(Nat.4) = Nat.1 or h.mod(Nat.4) = Nat.2)
        mod_lt(h, Nat.4)
        h.mod(Nat.4) < Nat.4
        nat_lt_four_cases_local(h.mod(Nat.4))
        if h.mod(Nat.4) = Nat.1 {
            false
        }
        if h.mod(Nat.4) = Nat.2 {
            false
        }
        h.mod(Nat.4) = Nat.0 or h.mod(Nat.4) = Nat.3
        Nat.2.divides((h * (h + Nat.1)).div(Nat.2)) =
            (h.mod(Nat.4) = Nat.0 or h.mod(Nat.4) = Nat.3)
    } else {
        if h.mod(Nat.4) = Nat.0 or h.mod(Nat.4) = Nat.3 {
            nat_half_prod_div_two_even_of_mod_four_zero_or_three(h)
            Nat.2.divides((h * (h + Nat.1)).div(Nat.2))
            false
        }
        not (h.mod(Nat.4) = Nat.0 or h.mod(Nat.4) = Nat.3)
        mod_lt(h, Nat.4)
        h.mod(Nat.4) < Nat.4
        nat_lt_four_cases_local(h.mod(Nat.4))
        if h.mod(Nat.4) = Nat.0 {
            false
        }
        if h.mod(Nat.4) = Nat.3 {
            false
        }
        h.mod(Nat.4) = Nat.1 or h.mod(Nat.4) = Nat.2
        Nat.2.divides((h * (h + Nat.1)).div(Nat.2)) =
            (h.mod(Nat.4) = Nat.0 or h.mod(Nat.4) = Nat.3)
    }
    (Nat.2.divides((h * (h + Nat.1)).div(Nat.2)) =
        (h.mod(Nat.4) = Nat.0 or h.mod(Nat.4) = Nat.3))
}

/// The second supplementary law in closed form:
/// (2/p) = (-1)^((p²-1)/8) = (-1)^(h(h+1)/2) for p = 2h + 1.
theorem prime_two_legendre_symbol_pow(p: Nat, h: Nat) {
    p.is_prime and p = Nat.2 * h + Nat.1 and
        (exists(g: Nat) { is_order_double_unit_generator_mod(g, p, h) })
    implies legendre_symbol(Nat.2, p) =
        (-Int.1).pow((h * (h + Nat.1)).div(Nat.2))
} by {
    if p.is_prime and p = Nat.2 * h + Nat.1 and
        (exists(g: Nat) { is_order_double_unit_generator_mod(g, p, h) }) {
        prime_two_legendre_symbol_mod_eight(p, h)
        legendre_symbol(Nat.2, p) =
            if p.congr_mod(Nat.1, Nat.8) or p.congr_mod(Nat.7, Nat.8) {
                Int.1
            } else {
                -Int.1
            }
        p.congr_mod(Nat.1, Nat.8) = (p.mod(Nat.8) = Nat.1.mod(Nat.8))
        lt_suc(Nat.1)
        Nat.1 < Nat.2
        lt_imp_lte_suc(Nat.1, Nat.2)
        Nat.2 <= Nat.3
        lt_suc(Nat.3)
        Nat.3 < Nat.4
        lt_suc(Nat.4)
        Nat.4 < Nat.5
        lt_suc(Nat.5)
        Nat.5 < Nat.6
        lt_suc(Nat.6)
        Nat.6 < Nat.7
        lt_suc(Nat.7)
        Nat.7 < Nat.8
        lt_trans(Nat.1, Nat.2, Nat.3)
        Nat.1 < Nat.3
        lt_trans(Nat.1, Nat.3, Nat.4)
        Nat.1 < Nat.4
        lt_trans(Nat.1, Nat.4, Nat.5)
        Nat.1 < Nat.5
        lt_trans(Nat.1, Nat.5, Nat.6)
        Nat.1 < Nat.6
        lt_trans(Nat.1, Nat.6, Nat.7)
        Nat.1 < Nat.7
        lt_trans(Nat.1, Nat.7, Nat.8)
        Nat.1 < Nat.8
        small_mod(Nat.1, Nat.8)
        Nat.1.mod(Nat.8) = Nat.1
        p.congr_mod(Nat.1, Nat.8) = (p.mod(Nat.8) = Nat.1)
        p.congr_mod(Nat.7, Nat.8) = (p.mod(Nat.8) = Nat.7.mod(Nat.8))
        lt_suc(Nat.7)
        Nat.7 < Nat.8
        small_mod(Nat.7, Nat.8)
        Nat.7.mod(Nat.8) = Nat.7
        p.congr_mod(Nat.7, Nat.8) = (p.mod(Nat.8) = Nat.7)
        p = Nat.2 * h + Nat.1
        p.mod(Nat.8) = (Nat.2 * h + Nat.1).mod(Nat.8)
        double_add_one_mod_eight_cases(h)
        ((Nat.2 * h + Nat.1).mod(Nat.8) = Nat.1 or
            (Nat.2 * h + Nat.1).mod(Nat.8) = Nat.7) =
            (h.mod(Nat.4) = Nat.0 or h.mod(Nat.4) = Nat.3)
        nat_half_prod_div_two_parity(h)
        Nat.2.divides((h * (h + Nat.1)).div(Nat.2)) =
            (h.mod(Nat.4) = Nat.0 or h.mod(Nat.4) = Nat.3)
        if p.congr_mod(Nat.1, Nat.8) or p.congr_mod(Nat.7, Nat.8) {
            if p.congr_mod(Nat.1, Nat.8) {
                p.mod(Nat.8) = Nat.1
                p.mod(Nat.8) = Nat.1 or p.mod(Nat.8) = Nat.7
            } else {
                p.congr_mod(Nat.7, Nat.8)
                p.mod(Nat.8) = Nat.7
                p.mod(Nat.8) = Nat.1 or p.mod(Nat.8) = Nat.7
            }
            if p.mod(Nat.8) = Nat.1 {
                (Nat.2 * h + Nat.1).mod(Nat.8) = Nat.1
                (Nat.2 * h + Nat.1).mod(Nat.8) = Nat.1 or
                    (Nat.2 * h + Nat.1).mod(Nat.8) = Nat.7
            } else {
                p.mod(Nat.8) = Nat.7
                (Nat.2 * h + Nat.1).mod(Nat.8) = Nat.7
                (Nat.2 * h + Nat.1).mod(Nat.8) = Nat.1 or
                    (Nat.2 * h + Nat.1).mod(Nat.8) = Nat.7
            }
            h.mod(Nat.4) = Nat.0 or h.mod(Nat.4) = Nat.3
            Nat.2.divides((h * (h + Nat.1)).div(Nat.2))
            int_neg_one_pow_parity((h * (h + Nat.1)).div(Nat.2))
            (-Int.1).pow((h * (h + Nat.1)).div(Nat.2)) = Int.1
            legendre_symbol(Nat.2, p) = Int.1
            legendre_symbol(Nat.2, p) =
                (-Int.1).pow((h * (h + Nat.1)).div(Nat.2))
        } else {
            not (p.congr_mod(Nat.1, Nat.8) or p.congr_mod(Nat.7, Nat.8))
            not (p.mod(Nat.8) = Nat.1 or p.mod(Nat.8) = Nat.7)
            if (Nat.2 * h + Nat.1).mod(Nat.8) = Nat.1 or
                (Nat.2 * h + Nat.1).mod(Nat.8) = Nat.7 {
                if (Nat.2 * h + Nat.1).mod(Nat.8) = Nat.1 {
                    p.mod(Nat.8) = Nat.1
                    false
                } else {
                    (Nat.2 * h + Nat.1).mod(Nat.8) = Nat.7
                    p.mod(Nat.8) = Nat.7
                    false
                }
                false
            }
            not ((Nat.2 * h + Nat.1).mod(Nat.8) = Nat.1 or
                (Nat.2 * h + Nat.1).mod(Nat.8) = Nat.7)
            not (h.mod(Nat.4) = Nat.0 or h.mod(Nat.4) = Nat.3)
            not Nat.2.divides((h * (h + Nat.1)).div(Nat.2))
            int_neg_one_pow_parity((h * (h + Nat.1)).div(Nat.2))
            (-Int.1).pow((h * (h + Nat.1)).div(Nat.2)) = -Int.1
            legendre_symbol(Nat.2, p) = -Int.1
            legendre_symbol(Nat.2, p) =
                (-Int.1).pow((h * (h + Nat.1)).div(Nat.2))
        }
        legendre_symbol(Nat.2, p) =
            (-Int.1).pow((h * (h + Nat.1)).div(Nat.2))
    }
}
