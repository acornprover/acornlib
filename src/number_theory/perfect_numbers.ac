from nat import Nat, divides_self, divides_trans, divides_sub, divides_lte,
    exp_ne_zero, exp_zero, exp_one, exp_add, add_sub, add_imp_sub, add_imp_sub_left,
    add_assoc, add_comm, add_zero_right, add_zero_left, add_one_right, add_one_left,
    suc_sub_one, mul_two_left, mul_comm, mul_assoc, gcd_divides_left, gcd_divides_right,
    gcd_of_prime, lte_imp_not_lt, lt_imp_lte_suc, lte_and_lt, lt_and_lte, lte_trans,
    lt_suc_right, lt_not_ref, not_lt_zero, sub_self, sub_zero,
    read_add_read, read_read_carry, nat_add_4_2, nat_add_4_3, nat_mul_2_6,
    nat_mul_3_4, nat_mul_7_8, nat_mul_8_7
from list import List, sum, map
from list import unique_same_contains_map_sum_eq
from number_theory.divisor_sum import nat_sigma, divisor_list, nat_sigma_prime,
    divisor_list_contains_implies, divisor_list_contains_of, divisor_list_is_unique,
    divisors_up_to, divisors_up_to_suc_yes, divisors_up_to_suc_no, divisors_up_to_one,
    one_divides_nat, sum_map_nat_identity_arithmetic_fn_eq_sum
from number_theory.liouville import divisor_list_six, divisor_list_four
from number_theory.coprime import nat_divides_one_imp_one, coprime_comm
from number_theory.zsigmondy import seven_is_prime, not_divides_of_lt, lt_ne, one_le_two_pow,
    two_divides_two_pow, lt_three_five, lt_zero_three
from number_theory.factorisation import count_prime_factor, divides_imp_count_prime_factor_le,
    count_prime_factor_pow, count_prime_factor_pow_other, count_prime_factor_ext
from number_theory.arithmetic_functions import is_multiplicative_nat_fn,
    multiplicative_nat_fn_apply, nat_identity_arithmetic_fn
from number_theory.dirichlet import nat_sigma_multiplicative
from data.nat.nat_binary_digits import two_pow_positive, two_pow_suc
numerals Nat

/// A natural number `n` is perfect when the sum of its positive divisors is
/// `2 n` (the divisors sum to twice the number itself).
define is_perfect(n: Nat) -> Bool {
    nat_sigma(n) = Nat.2 * n
}

// ---------------------------------------------------------------------------
// The first perfect number: 6.
// ---------------------------------------------------------------------------

/// Six is the sum of its positive divisors: `sigma(6) = 12`.
theorem nat_sigma_six {
    nat_sigma(Nat.6) = Nat.12
} by {
    divisor_list_six
    divisor_list(Nat.6) =
        List.cons(Nat.6, List.cons(Nat.3, List.cons(Nat.2, List.cons(Nat.1, List.nil[Nat]))))
    nat_sigma(Nat.6) = sum(divisor_list(Nat.6))
    sum(List.cons(Nat.6, List.cons(Nat.3, List.cons(Nat.2, List.cons(Nat.1, List.nil[Nat]))))) =
        Nat.6 + sum(List.cons(Nat.3, List.cons(Nat.2, List.cons(Nat.1, List.nil[Nat]))))
    sum(List.cons(Nat.3, List.cons(Nat.2, List.cons(Nat.1, List.nil[Nat])))) =
        Nat.3 + sum(List.cons(Nat.2, List.cons(Nat.1, List.nil[Nat])))
    sum(List.cons(Nat.2, List.cons(Nat.1, List.nil[Nat]))) =
        Nat.2 + sum(List.cons(Nat.1, List.nil[Nat]))
    sum(List.cons(Nat.1, List.nil[Nat])) = Nat.1 + sum(List.nil[Nat])
    sum(List.nil[Nat]) = Nat.0
    Nat.1 + Nat.0 = Nat.1
    Nat.2 + Nat.1 = Nat.3
    Nat.3 + Nat.3 = Nat.6
    Nat.6 + Nat.6 = Nat.12
    nat_sigma(Nat.6) = Nat.12
}

/// Six is perfect.
theorem six_is_perfect {
    is_perfect(Nat.6)
} by {
    is_perfect(Nat.6) = (nat_sigma(Nat.6) = Nat.2 * Nat.6)
    nat_sigma_six
    nat_sigma(Nat.6) = Nat.12
    Nat.2 * Nat.6 = Nat.12
    nat_sigma(Nat.6) = Nat.2 * Nat.6
    is_perfect(Nat.6)
}

// ---------------------------------------------------------------------------
// The second perfect number: 28 = 4 * 7, with 4 and 7 coprime.
// ---------------------------------------------------------------------------

/// Four is coprime with seven, since seven is prime and does not divide four.
theorem four_coprime_seven {
    Nat.4.coprime(Nat.7)
} by {
    seven_is_prime
    Nat.7.is_prime
    gcd_of_prime(Nat.7, Nat.4)
    if Nat.7.gcd(Nat.4) = Nat.1 {
        Nat.7.coprime(Nat.4)
        coprime_comm(Nat.7, Nat.4)
        Nat.4.coprime(Nat.7)
    } else {
        Nat.7.divides(Nat.4)
        Nat.0 < Nat.4
        Nat.4 + Nat.3 = Nat.7
        exists(c: Nat) { Nat.4 + c = Nat.7 }
        Nat.4 <= Nat.7
        Nat.3 != Nat.0
        lt_ne(Nat.4, Nat.7, Nat.3)
        Nat.4 != Nat.7
        Nat.4 < Nat.7
        not_divides_of_lt(Nat.7, Nat.4)
        false
    }
}

/// `sigma(4) = 7`: the divisors of four are `1, 2, 4`.
theorem nat_sigma_four {
    nat_sigma(Nat.4) = Nat.7
} by {
    divisor_list_four
    divisor_list(Nat.4) = List.cons(Nat.4, List.cons(Nat.2, List.cons(Nat.1, List.nil[Nat])))
    nat_sigma(Nat.4) = sum(divisor_list(Nat.4))
    sum(List.cons(Nat.4, List.cons(Nat.2, List.cons(Nat.1, List.nil[Nat])))) =
        Nat.4 + sum(List.cons(Nat.2, List.cons(Nat.1, List.nil[Nat])))
    sum(List.cons(Nat.2, List.cons(Nat.1, List.nil[Nat]))) =
        Nat.2 + sum(List.cons(Nat.1, List.nil[Nat]))
    sum(List.cons(Nat.1, List.nil[Nat])) = Nat.1 + sum(List.nil[Nat])
    sum(List.nil[Nat]) = Nat.0
    Nat.1 + Nat.0 = Nat.1
    Nat.2 + Nat.1 = Nat.3
    Nat.4 + Nat.3 = Nat.7
    nat_sigma(Nat.4) = Nat.7
}

/// `sigma(7) = 8`, since seven is prime.
theorem nat_sigma_seven {
    nat_sigma(Nat.7) = Nat.8
} by {
    seven_is_prime
    Nat.7.is_prime
    nat_sigma_prime(Nat.7)
    nat_sigma(Nat.7) = Nat.7 + Nat.1
    Nat.7 + Nat.1 = Nat.8
    nat_sigma(Nat.7) = Nat.8
}

/// `sigma(28) = sigma(4) sigma(7) = 7 * 8 = 56`, by multiplicativity.
theorem nat_sigma_28 {
    nat_sigma(Nat.28) = Nat.56
} by {
    Nat.4 * Nat.7 = Nat.28
    four_coprime_seven
    Nat.4.coprime(Nat.7)
    nat_sigma_multiplicative
    is_multiplicative_nat_fn(nat_sigma)
    multiplicative_nat_fn_apply(nat_sigma, Nat.4, Nat.7)
    nat_sigma(Nat.4 * Nat.7) = nat_sigma(Nat.4) * nat_sigma(Nat.7)
    nat_sigma_four
    nat_sigma(Nat.4) = Nat.7
    nat_sigma_seven
    nat_sigma(Nat.7) = Nat.8
    nat_sigma(Nat.4 * Nat.7) = Nat.7 * Nat.8
    nat_mul_7_8
    Nat.7 * Nat.8 = Nat.56
    nat_sigma(Nat.4 * Nat.7) = Nat.56
    Nat.4 * Nat.7 = Nat.28
    nat_sigma(Nat.28) = Nat.56
}

/// `28 + 28 = 56`, by decimal arithmetic.
theorem twenty_eight_plus_twenty_eight {
    Nat.28 + Nat.28 = Nat.56
} by {
    Nat.28 = Nat.2.read(Nat.8)
    read_add_read(Nat.2, Nat.8, Nat.2, Nat.8)
    Nat.2.read(Nat.8) + Nat.2.read(Nat.8) = (Nat.2 + Nat.2).read(Nat.8 + Nat.8)
    Nat.2 + Nat.2 = Nat.4
    Nat.8 + Nat.8 = Nat.16
    (Nat.2 + Nat.2).read(Nat.8 + Nat.8) = Nat.4.read(Nat.16)
    Nat.4.read(Nat.16) = Nat.4.read(Nat.10 * Nat.1 + Nat.6)
    read_read_carry(Nat.4, Nat.1, Nat.6)
    Nat.4.read(Nat.10 * Nat.1 + Nat.6) = (Nat.4 + Nat.1).read(Nat.6)
    Nat.4 + Nat.1 = Nat.5
    (Nat.4 + Nat.1).read(Nat.6) = Nat.5.read(Nat.6)
    Nat.5.read(Nat.6) = Nat.56
    Nat.28 + Nat.28 = Nat.56
}

/// Twice twenty-eight is fifty-six.
theorem two_mul_28_eq_56 {
    Nat.2 * Nat.28 = Nat.56
} by {
    mul_two_left(Nat.28)
    Nat.2 * Nat.28 = Nat.28 + Nat.28
    twenty_eight_plus_twenty_eight
    Nat.28 + Nat.28 = Nat.56
    Nat.2 * Nat.28 = Nat.56
}

/// Twenty-eight is perfect.
theorem twenty_eight_is_perfect {
    is_perfect(Nat.28)
} by {
    is_perfect(Nat.28) = (nat_sigma(Nat.28) = Nat.2 * Nat.28)
    nat_sigma_28
    nat_sigma(Nat.28) = Nat.56
    two_mul_28_eq_56
    Nat.2 * Nat.28 = Nat.56
    nat_sigma(Nat.28) = Nat.2 * Nat.28
    is_perfect(Nat.28)
}
