from data.basic.functions import binary_function_extensionality, compose, function_extensionality
from pair.base import Pair, curry, pair_assoc_left, pair_assoc_left_first,
    pair_assoc_left_second, pair_assoc_right, pair_assoc_right_first,
    pair_assoc_right_second, pair_ext, pair_map, pair_map_first_eq,
    pair_map_second_eq, pair_new_first, pair_new_second, uncurry

/// Left association is natural for componentwise maps of nested pairs.
theorem pair_assoc_left_map_natural[A, B, C, D, E, F](
    f: A -> D, g: B -> E, h: C -> F, p: Pair[A, Pair[B, C]]) {
    pair_assoc_left(pair_map(f, pair_map(g, h), p)) =
        pair_map(pair_map(f, g), h, pair_assoc_left(p))
} by {
    let mapped = pair_map(f, pair_map(g, h), p)
    let lhs = pair_assoc_left(mapped)
    let assoc = pair_assoc_left(p)
    let rhs = pair_map(pair_map(f, g), h, assoc)

    pair_assoc_left_first(mapped)
    lhs.first = Pair.new(mapped.first, mapped.second.first)
    pair_map_first_eq(f, pair_map(g, h), p)
    mapped.first = f(p.first)
    pair_map_second_eq(f, pair_map(g, h), p)
    mapped.second = pair_map(g, h, p.second)
    mapped.second.first = pair_map(g, h, p.second).first
    pair_map_first_eq(g, h, p.second)
    pair_map(g, h, p.second).first = g(p.second.first)
    mapped.second.first = g(p.second.first)
    lhs.first = Pair.new(f(p.first), g(p.second.first))

    pair_map_first_eq(pair_map(f, g), h, assoc)
    rhs.first = pair_map(f, g, assoc.first)
    pair_assoc_left_first(p)
    assoc.first = Pair.new(p.first, p.second.first)
    rhs.first = pair_map(f, g, Pair.new(p.first, p.second.first))
    pair_map_first_eq(f, g, Pair.new(p.first, p.second.first))
    pair_map(f, g, Pair.new(p.first, p.second.first)).first =
        f(Pair.new(p.first, p.second.first).first)
    pair_new_first(p.first, p.second.first)
    Pair.new(p.first, p.second.first).first = p.first
    pair_map(f, g, Pair.new(p.first, p.second.first)).first = f(p.first)
    pair_map_second_eq(f, g, Pair.new(p.first, p.second.first))
    pair_map(f, g, Pair.new(p.first, p.second.first)).second =
        g(Pair.new(p.first, p.second.first).second)
    pair_new_second(p.first, p.second.first)
    Pair.new(p.first, p.second.first).second = p.second.first
    pair_map(f, g, Pair.new(p.first, p.second.first)).second = g(p.second.first)
    pair_ext(pair_map(f, g, Pair.new(p.first, p.second.first)),
        Pair.new(f(p.first), g(p.second.first)))
    pair_map(f, g, Pair.new(p.first, p.second.first)) =
        Pair.new(f(p.first), g(p.second.first))
    rhs.first = Pair.new(f(p.first), g(p.second.first))
    lhs.first = rhs.first

    pair_assoc_left_second(mapped)
    lhs.second = mapped.second.second
    mapped.second.second = pair_map(g, h, p.second).second
    pair_map_second_eq(g, h, p.second)
    pair_map(g, h, p.second).second = h(p.second.second)
    lhs.second = h(p.second.second)

    pair_map_second_eq(pair_map(f, g), h, assoc)
    rhs.second = h(assoc.second)
    pair_assoc_left_second(p)
    assoc.second = p.second.second
    rhs.second = h(p.second.second)
    lhs.second = rhs.second

    pair_ext(lhs, rhs)
}

/// Right association is natural for componentwise maps of nested pairs.
theorem pair_assoc_right_map_natural[A, B, C, D, E, F](
    f: A -> D, g: B -> E, h: C -> F, p: Pair[Pair[A, B], C]) {
    pair_assoc_right(pair_map(pair_map(f, g), h, p)) =
        pair_map(f, pair_map(g, h), pair_assoc_right(p))
} by {
    let mapped = pair_map(pair_map(f, g), h, p)
    let lhs = pair_assoc_right(mapped)
    let assoc = pair_assoc_right(p)
    let rhs = pair_map(f, pair_map(g, h), assoc)

    pair_assoc_right_first(mapped)
    lhs.first = mapped.first.first
    pair_map_first_eq(pair_map(f, g), h, p)
    mapped.first = pair_map(f, g, p.first)
    mapped.first.first = pair_map(f, g, p.first).first
    pair_map_first_eq(f, g, p.first)
    pair_map(f, g, p.first).first = f(p.first.first)
    lhs.first = f(p.first.first)

    pair_map_first_eq(f, pair_map(g, h), assoc)
    rhs.first = f(assoc.first)
    pair_assoc_right_first(p)
    assoc.first = p.first.first
    rhs.first = f(p.first.first)
    lhs.first = rhs.first

    pair_assoc_right_second(mapped)
    lhs.second = Pair.new(mapped.first.second, mapped.second)
    mapped.first.second = pair_map(f, g, p.first).second
    pair_map_second_eq(f, g, p.first)
    pair_map(f, g, p.first).second = g(p.first.second)
    mapped.first.second = g(p.first.second)
    pair_map_second_eq(pair_map(f, g), h, p)
    mapped.second = h(p.second)
    lhs.second = Pair.new(g(p.first.second), h(p.second))

    pair_map_second_eq(f, pair_map(g, h), assoc)
    rhs.second = pair_map(g, h, assoc.second)
    pair_assoc_right_second(p)
    assoc.second = Pair.new(p.first.second, p.second)
    rhs.second = pair_map(g, h, Pair.new(p.first.second, p.second))
    pair_map_first_eq(g, h, Pair.new(p.first.second, p.second))
    pair_map(g, h, Pair.new(p.first.second, p.second)).first =
        g(Pair.new(p.first.second, p.second).first)
    pair_new_first(p.first.second, p.second)
    Pair.new(p.first.second, p.second).first = p.first.second
    pair_map(g, h, Pair.new(p.first.second, p.second)).first = g(p.first.second)
    pair_map_second_eq(g, h, Pair.new(p.first.second, p.second))
    pair_map(g, h, Pair.new(p.first.second, p.second)).second =
        h(Pair.new(p.first.second, p.second).second)
    pair_new_second(p.first.second, p.second)
    Pair.new(p.first.second, p.second).second = p.second
    pair_map(g, h, Pair.new(p.first.second, p.second)).second = h(p.second)
    pair_ext(pair_map(g, h, Pair.new(p.first.second, p.second)),
        Pair.new(g(p.first.second), h(p.second)))
    pair_map(g, h, Pair.new(p.first.second, p.second)) =
        Pair.new(g(p.first.second), h(p.second))
    rhs.second = Pair.new(g(p.first.second), h(p.second))
    lhs.second = rhs.second

    pair_ext(lhs, rhs)
}

/// Composing the output of a binary function commutes with currying.
theorem curry_compose_output[T, U, V, W](f: (T, U) -> V, h: V -> W) {
    curry(compose(h, uncurry(f))) = function(x: T, y: U) { h(f(x, y)) }
} by {
    let rhs = function(x: T, y: U) { h(f(x, y)) }
    forall(x: T, y: U) {
        curry(compose(h, uncurry(f)), x, y) = compose(h, uncurry(f), Pair.new(x, y))
        compose(h, uncurry(f), Pair.new(x, y)) = h(uncurry(f, Pair.new(x, y)))
        uncurry(f, Pair.new(x, y)) = f(Pair.new(x, y).first, Pair.new(x, y).second)
        pair_new_first(x, y)
        Pair.new(x, y).first = x
        pair_new_second(x, y)
        Pair.new(x, y).second = y
        f(Pair.new(x, y).first, Pair.new(x, y).second) = f(x, y)
        uncurry(f, Pair.new(x, y)) = f(x, y)
        h(uncurry(f, Pair.new(x, y))) = h(f(x, y))
        rhs(x, y) = h(f(x, y))
        curry(compose(h, uncurry(f)), x, y) = rhs(x, y)
    }
    binary_function_extensionality(curry(compose(h, uncurry(f))), rhs)
}

/// Output composition commutes with uncurrying a curried pair function.
theorem uncurry_curry_compose_input[T, U, V, W](f: Pair[T, U] -> V, h: V -> W) {
    uncurry(function(x: T, y: U) { h(curry(f, x, y)) }) = compose(h, f)
} by {
    let lhs = uncurry(function(x: T, y: U) { h(curry(f, x, y)) })
    let rhs = compose(h, f)
    forall(p: Pair[T, U]) {
        lhs(p) = function(x: T, y: U) { h(curry(f, x, y)) }(p.first, p.second)
        function(x: T, y: U) { h(curry(f, x, y)) }(p.first, p.second) =
            h(curry(f, p.first, p.second))
        curry(f, p.first, p.second) = f(Pair.new(p.first, p.second))
        h(curry(f, p.first, p.second)) = h(f(Pair.new(p.first, p.second)))
        Pair.new(p.first, p.second) = p
        h(f(Pair.new(p.first, p.second))) = h(f(p))
        rhs(p) = h(f(p))
        lhs(p) = rhs(p)
    }
    function_extensionality(lhs, rhs)
}
