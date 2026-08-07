from data.basic.functions import compose, function_extensionality, identity_fn
from pair.base import Pair, fan_out, fan_out_first, fan_out_second, pair_ext,
    pair_first, pair_second, pair_map, pair_map_first, pair_map_second,
    pair_map_first_eq, pair_map_second_eq, pair_new_first, pair_new_second,
    swap_first, swap_second

/// Mapping only the first component is componentwise pair mapping with identity on the second.
theorem pair_map_first_eq_map[T, U, V](f: T -> V, p: Pair[T, U]) {
    pair_map_first(f, p) = pair_map(f, identity_fn[U], p)
} by {
    let lhs = pair_map_first(f, p)
    let rhs = pair_map(f, identity_fn[U], p)
    lhs.first = f(p.first)
    rhs.first = f(p.first)
    lhs.first = rhs.first
    lhs.second = p.second
    rhs.second = identity_fn[U](p.second)
    identity_fn[U](p.second) = p.second
    rhs.second = p.second
    lhs.second = rhs.second
    pair_ext(lhs, rhs)
}

/// Mapping only the second component is componentwise pair mapping with identity on the first.
theorem pair_map_second_eq_map[T, U, V](g: U -> V, p: Pair[T, U]) {
    pair_map_second(g, p) = pair_map(identity_fn[T], g, p)
} by {
    let lhs = pair_map_second(g, p)
    let rhs = pair_map(identity_fn[T], g, p)
    lhs.first = p.first
    rhs.first = identity_fn[T](p.first)
    identity_fn[T](p.first) = p.first
    rhs.first = p.first
    lhs.first = rhs.first
    lhs.second = g(p.second)
    rhs.second = g(p.second)
    lhs.second = rhs.second
    pair_ext(lhs, rhs)
}

/// Swapping after componentwise mapping is componentwise mapping in the swapped order.
theorem pair_swap_map[T, U, V, W](f: T -> V, g: U -> W, p: Pair[T, U]) {
    pair_map(f, g, p).swap = pair_map(g, f, p.swap)
} by {
    let q = pair_map(f, g, p)
    let rhs = pair_map(g, f, p.swap)

    swap_first(q)
    q.swap.first = q.second
    pair_map_second_eq(f, g, p)
    q.second = g(p.second)
    q.swap.first = g(p.second)
    swap_first(p)
    p.swap.first = p.second
    rhs.first = g(p.swap.first)
    rhs.first = g(p.second)
    q.swap.first = rhs.first

    swap_second(q)
    q.swap.second = q.first
    pair_map_first_eq(f, g, p)
    q.first = f(p.first)
    q.swap.second = f(p.first)
    swap_second(p)
    p.swap.second = p.first
    rhs.second = f(p.swap.second)
    rhs.second = f(p.first)
    q.swap.second = rhs.second

    pair_ext(q.swap, rhs)
}

/// Mapping the first component and swapping is mapping the second component after swapping.
theorem pair_map_first_swap[T, U, V](f: T -> V, p: Pair[T, U]) {
    pair_map_first(f, p).swap = pair_map_second(f, p.swap)
} by {
    let q = pair_map_first(f, p)
    let rhs = pair_map_second(f, p.swap)

    swap_first(q)
    q.swap.first = q.second
    q.second = p.second
    q.swap.first = p.second
    swap_first(p)
    p.swap.first = p.second
    rhs.first = p.swap.first
    rhs.first = p.second
    q.swap.first = rhs.first

    swap_second(q)
    q.swap.second = q.first
    q.first = f(p.first)
    q.swap.second = f(p.first)
    swap_second(p)
    p.swap.second = p.first
    rhs.second = f(p.swap.second)
    rhs.second = f(p.first)
    q.swap.second = rhs.second

    pair_ext(q.swap, rhs)
}

/// Mapping the second component and swapping is mapping the first component after swapping.
theorem pair_map_second_swap[T, U, V](g: U -> V, p: Pair[T, U]) {
    pair_map_second(g, p).swap = pair_map_first(g, p.swap)
} by {
    let q = pair_map_second(g, p)
    let rhs = pair_map_first(g, p.swap)

    swap_first(q)
    q.swap.first = q.second
    q.second = g(p.second)
    q.swap.first = g(p.second)
    swap_first(p)
    p.swap.first = p.second
    rhs.first = g(p.swap.first)
    rhs.first = g(p.second)
    q.swap.first = rhs.first

    swap_second(q)
    q.swap.second = q.first
    q.first = p.first
    q.swap.second = p.first
    swap_second(p)
    p.swap.second = p.first
    rhs.second = p.swap.second
    rhs.second = p.first
    q.swap.second = rhs.second

    pair_ext(q.swap, rhs)
}

/// A fan-out from one value is the componentwise image of its diagonal pair.
theorem fan_out_eq_pair_map_diagonal[Z, X, Y](f: Z -> X, g: Z -> Y, z: Z) {
    fan_out(f, g, z) = pair_map(f, g, Pair.new(z, z))
} by {
    let diagonal = Pair.new(z, z)
    let lhs = fan_out(f, g, z)
    let rhs = pair_map(f, g, diagonal)

    fan_out_first(f, g, z)
    lhs.first = f(z)
    pair_map_first_eq(f, g, diagonal)
    rhs.first = f(diagonal.first)
    pair_new_first(z, z)
    diagonal.first = z
    rhs.first = f(z)
    lhs.first = rhs.first

    fan_out_second(f, g, z)
    lhs.second = g(z)
    pair_map_second_eq(f, g, diagonal)
    rhs.second = g(diagonal.second)
    pair_new_second(z, z)
    diagonal.second = z
    rhs.second = g(z)
    lhs.second = rhs.second

    pair_ext(lhs, rhs)
}

/// Swapping a fan-out exchanges its two component maps.
theorem fan_out_swap[Z, X, Y](f: Z -> X, g: Z -> Y, z: Z) {
    fan_out(f, g, z).swap = fan_out(g, f, z)
} by {
    let q = fan_out(f, g, z)
    let rhs = fan_out(g, f, z)

    swap_first(q)
    q.swap.first = q.second
    fan_out_second(f, g, z)
    q.second = g(z)
    q.swap.first = g(z)
    fan_out_first(g, f, z)
    rhs.first = g(z)
    q.swap.first = rhs.first

    swap_second(q)
    q.swap.second = q.first
    fan_out_first(f, g, z)
    q.first = f(z)
    q.swap.second = f(z)
    fan_out_second(g, f, z)
    rhs.second = f(z)
    q.swap.second = rhs.second

    pair_ext(q.swap, rhs)
}

/// Projecting the first component after pair mapping is function composition with first projection.
theorem pair_first_compose_pair_map[T, U, V, W](f: T -> V, g: U -> W) {
    compose(pair_first[V, W], pair_map[T, U, V, W](f, g)) = compose(f, pair_first[T, U])
} by {
    forall(p: Pair[T, U]) {
        compose(pair_first[V, W], pair_map[T, U, V, W](f, g), p) = pair_first[V, W](pair_map(f, g, p))
        pair_first[V, W](pair_map(f, g, p)) = pair_map(f, g, p).first
        pair_map_first_eq(f, g, p)
        pair_map(f, g, p).first = f(p.first)
        compose(f, pair_first[T, U], p) = f(pair_first[T, U](p))
        pair_first[T, U](p) = p.first
        compose(f, pair_first[T, U], p) = f(p.first)
        compose(pair_first[V, W], pair_map[T, U, V, W](f, g), p) = compose(f, pair_first[T, U], p)
    }
    function_extensionality(compose(pair_first[V, W], pair_map[T, U, V, W](f, g)),
        compose(f, pair_first[T, U]))
}

/// Projecting the second component after pair mapping is function composition with second projection.
theorem pair_second_compose_pair_map[T, U, V, W](f: T -> V, g: U -> W) {
    compose(pair_second[V, W], pair_map[T, U, V, W](f, g)) = compose(g, pair_second[T, U])
} by {
    forall(p: Pair[T, U]) {
        compose(pair_second[V, W], pair_map[T, U, V, W](f, g), p) = pair_second[V, W](pair_map(f, g, p))
        pair_second[V, W](pair_map(f, g, p)) = pair_map(f, g, p).second
        pair_map_second_eq(f, g, p)
        pair_map(f, g, p).second = g(p.second)
        compose(g, pair_second[T, U], p) = g(pair_second[T, U](p))
        pair_second[T, U](p) = p.second
        compose(g, pair_second[T, U], p) = g(p.second)
        compose(pair_second[V, W], pair_map[T, U, V, W](f, g), p) = compose(g, pair_second[T, U], p)
    }
    function_extensionality(compose(pair_second[V, W], pair_map[T, U, V, W](f, g)),
        compose(g, pair_second[T, U]))
}

/// The first projection composed with fan-out recovers the first component map.
theorem pair_first_compose_fan_out[Z, X, Y](f: Z -> X, g: Z -> Y) {
    compose(pair_first[X, Y], fan_out[Z, X, Y](f, g)) = f
} by {
    forall(z: Z) {
        compose(pair_first[X, Y], fan_out[Z, X, Y](f, g), z) = pair_first[X, Y](fan_out(f, g, z))
        pair_first[X, Y](fan_out(f, g, z)) = fan_out(f, g, z).first
        fan_out_first(f, g, z)
        fan_out(f, g, z).first = f(z)
        compose(pair_first[X, Y], fan_out[Z, X, Y](f, g), z) = f(z)
    }
    function_extensionality(compose(pair_first[X, Y], fan_out[Z, X, Y](f, g)), f)
}

/// The second projection composed with fan-out recovers the second component map.
theorem pair_second_compose_fan_out[Z, X, Y](f: Z -> X, g: Z -> Y) {
    compose(pair_second[X, Y], fan_out[Z, X, Y](f, g)) = g
} by {
    forall(z: Z) {
        compose(pair_second[X, Y], fan_out[Z, X, Y](f, g), z) = pair_second[X, Y](fan_out(f, g, z))
        pair_second[X, Y](fan_out(f, g, z)) = fan_out(f, g, z).second
        fan_out_second(f, g, z)
        fan_out(f, g, z).second = g(z)
        compose(pair_second[X, Y], fan_out[Z, X, Y](f, g), z) = g(z)
    }
    function_extensionality(compose(pair_second[X, Y], fan_out[Z, X, Y](f, g)), g)
}

/// Pair mapping is fan-out of the two mapped projections.
theorem pair_map_eq_fan_out_projections[T, U, V, W](f: T -> V, g: U -> W, p: Pair[T, U]) {
    pair_map(f, g, p) = fan_out(compose(f, pair_first[T, U]), compose(g, pair_second[T, U]), p)
} by {
    let lhs = pair_map(f, g, p)
    let rhs = fan_out(compose(f, pair_first[T, U]), compose(g, pair_second[T, U]), p)

    pair_map_first_eq(f, g, p)
    lhs.first = f(p.first)
    fan_out_first(compose(f, pair_first[T, U]), compose(g, pair_second[T, U]), p)
    rhs.first = compose(f, pair_first[T, U], p)
    compose(f, pair_first[T, U], p) = f(pair_first[T, U](p))
    pair_first[T, U](p) = p.first
    rhs.first = f(p.first)
    lhs.first = rhs.first

    pair_map_second_eq(f, g, p)
    lhs.second = g(p.second)
    fan_out_second(compose(f, pair_first[T, U]), compose(g, pair_second[T, U]), p)
    rhs.second = compose(g, pair_second[T, U], p)
    compose(g, pair_second[T, U], p) = g(pair_second[T, U](p))
    pair_second[T, U](p) = p.second
    rhs.second = g(p.second)
    lhs.second = rhs.second

    pair_ext(lhs, rhs)
}
