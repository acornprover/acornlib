from data.basic.functions import compose, identity_fn

/// An ordered pair of two values, possibly of different types.
structure Pair[T, U] {
    /// The first element of the pair.
    first: T
    /// The second element of the pair.
    second: U
}

/// The first projection of a pair.
define pair_first[T, U](p: Pair[T, U]) -> T {
    p.first
}

/// The second projection of a pair.
define pair_second[T, U](p: Pair[T, U]) -> U {
    p.second
}

/// The image of a pair under componentwise maps.
define pair_map[T, U, V, W](f: T -> V, g: U -> W, p: Pair[T, U]) -> Pair[V, W] {
    Pair.new(f(p.first), g(p.second))
}

/// The map into a product determined by two maps out of a common source:
/// `z` is sent to the pair of its images under `f` and `g`.
define fan_out[Z, X, Y](f: Z -> X, g: Z -> Y, z: Z) -> Pair[X, Y] {
    Pair.new(f(z), g(z))
}

/// The first projection of `fan_out` is the first component function.
theorem fan_out_first[Z, X, Y](f: Z -> X, g: Z -> Y, z: Z) {
    fan_out(f, g, z).first = f(z)
}

/// The second projection of `fan_out` is the second component function.
theorem fan_out_second[Z, X, Y](f: Z -> X, g: Z -> Y, z: Z) {
    fan_out(f, g, z).second = g(z)
}

/// The image of a pair under a map on the first component.
define pair_map_first[T, U, V](f: T -> V, p: Pair[T, U]) -> Pair[V, U] {
    Pair.new(f(p.first), p.second)
}

/// The image of a pair under a map on the second component.
define pair_map_second[T, U, V](f: U -> V, p: Pair[T, U]) -> Pair[T, V] {
    Pair.new(p.first, f(p.second))
}

/// Left association of a nested pair.
define pair_assoc_left[T, U, V](p: Pair[T, Pair[U, V]]) -> Pair[Pair[T, U], V] {
    Pair.new(Pair.new(p.first, p.second.first), p.second.second)
}

/// Right association of a nested pair.
define pair_assoc_right[T, U, V](p: Pair[Pair[T, U], V]) -> Pair[T, Pair[U, V]] {
    Pair.new(p.first.first, Pair.new(p.first.second, p.second))
}

/// Currying changes a function on pairs into a binary function.
define curry[T, U, V](f: Pair[T, U] -> V, x: T, y: U) -> V {
    f(Pair.new(x, y))
}

/// Uncurrying changes a binary function into a function on pairs.
define uncurry[T, U, V](f: (T, U) -> V, p: Pair[T, U]) -> V {
    f(p.first, p.second)
}

attributes Pair[T, U] {
    /// The first projection.
    let fst: Pair[T, U] -> T = pair_first

    /// The second projection.
    let snd: Pair[T, U] -> U = pair_second

    /// Swaps the first and second elements of the pair.
    define swap(self) -> Pair[U, T] {
        Pair.new(self.second, self.first)
    }

    /// Applies functions to both components.
    define map[V, W](self, f: T -> V, g: U -> W) -> Pair[V, W] {
        pair_map(f, g, self)
    }

    /// Applies a function to the first component.
    define map_first[V](self, f: T -> V) -> Pair[V, U] {
        pair_map_first(f, self)
    }

    /// Applies a function to the second component.
    define map_second[V](self, f: U -> V) -> Pair[T, V] {
        pair_map_second(f, self)
    }
}

/// A pair is determined by its two projections.
theorem pair_ext[T, U](p: Pair[T, U], q: Pair[T, U]) {
    p.first = q.first and p.second = q.second implies p = q
}

/// Rebuilding a pair from its projections gives the original pair.
theorem pair_eta[T, U](p: Pair[T, U]) {
    Pair.new(p.first, p.second) = p
}

/// The first projection of a new pair is its first component.
theorem pair_new_first[T, U](x: T, y: U) {
    Pair.new(x, y).first = x
}

/// The second projection of a new pair is its second component.
theorem pair_new_second[T, U](x: T, y: U) {
    Pair.new(x, y).second = y
}

/// Swapping a pair moves the second component to the first projection.
theorem swap_first[T, U](p: Pair[T, U]) {
    p.swap.first = p.second
}

/// Swapping a pair moves the first component to the second projection.
theorem swap_second[T, U](p: Pair[T, U]) {
    p.swap.second = p.first
}

/// Swapping twice gives the original pair.
theorem swap_swap[T, U](p: Pair[T, U]) {
    p.swap.swap = p
}

/// Swapping pairs preserves and reflects equality.
theorem swap_eq_swap[T, U](p: Pair[T, U], q: Pair[T, U]) {
    p.swap = q.swap implies p = q
}

/// Componentwise pair maps have the expected first projection.
theorem pair_map_first_eq[T, U, V, W](f: T -> V, g: U -> W, p: Pair[T, U]) {
    pair_map(f, g, p).first = f(p.first)
}

/// Componentwise pair maps have the expected second projection.
theorem pair_map_second_eq[T, U, V, W](f: T -> V, g: U -> W, p: Pair[T, U]) {
    pair_map(f, g, p).second = g(p.second)
}

/// Mapping both components is the same as mapping the first and then the second.
theorem pair_map_eq_map_second_map_first[T, U, V, W](f: T -> V, g: U -> W, p: Pair[T, U]) {
    pair_map(f, g, p) = pair_map_second(g, pair_map_first(f, p))
}

/// Mapping both components by identity gives the original pair.
theorem pair_map_identity[T, U](p: Pair[T, U]) {
    pair_map(identity_fn[T], identity_fn[U], p) = p
}

/// Componentwise pair maps compose componentwise.
theorem pair_map_comp[A, B, C, D, E, F](f: C -> E, g: A -> C, h: D -> F, i: B -> D,
    p: Pair[A, B]) {
    pair_map(compose(f, g), compose(h, i), p) = pair_map(f, h, pair_map(g, i, p))
}

/// Mapping the first component by identity gives the original pair.
theorem pair_map_first_identity[T, U](p: Pair[T, U]) {
    pair_map_first(identity_fn[T], p) = p
}

/// Mapping the second component by identity gives the original pair.
theorem pair_map_second_identity[T, U](p: Pair[T, U]) {
    pair_map_second(identity_fn[U], p) = p
}

/// Maps on the first component compose as functions do.
theorem pair_map_first_comp[A, B, C, D](f: C -> D, g: A -> C, p: Pair[A, B]) {
    pair_map_first(compose(f, g), p) = pair_map_first(f, pair_map_first(g, p))
}

/// Maps on the second component compose as functions do.
theorem pair_map_second_comp[A, B, C, D](f: C -> D, g: B -> C, p: Pair[A, B]) {
    pair_map_second(compose(f, g), p) = pair_map_second(f, pair_map_second(g, p))
}

/// Left association has the expected first projection.
theorem pair_assoc_left_first[T, U, V](p: Pair[T, Pair[U, V]]) {
    pair_assoc_left(p).first = Pair.new(p.first, p.second.first)
}

/// Left association has the expected second projection.
theorem pair_assoc_left_second[T, U, V](p: Pair[T, Pair[U, V]]) {
    pair_assoc_left(p).second = p.second.second
}

/// Right association has the expected first projection.
theorem pair_assoc_right_first[T, U, V](p: Pair[Pair[T, U], V]) {
    pair_assoc_right(p).first = p.first.first
}

/// Right association has the expected second projection.
theorem pair_assoc_right_second[T, U, V](p: Pair[Pair[T, U], V]) {
    pair_assoc_right(p).second = Pair.new(p.first.second, p.second)
}

/// Left association followed by right association gives the original nested pair.
theorem pair_assoc_right_left[T, U, V](p: Pair[T, Pair[U, V]]) {
    pair_assoc_right(pair_assoc_left(p)) = p
}

/// Right association followed by left association gives the original nested pair.
theorem pair_assoc_left_right[T, U, V](p: Pair[Pair[T, U], V]) {
    pair_assoc_left(pair_assoc_right(p)) = p
}

/// Currying after uncurrying recovers the original binary function.
theorem curry_uncurry[T, U, V](f: (T, U) -> V) {
    curry(uncurry(f)) = f
}

/// Uncurrying after currying recovers the original function on pairs.
theorem uncurry_curry[T, U, V](f: Pair[T, U] -> V) {
    uncurry(curry(f)) = f
}
