from data.basic.functions import is_injective_fn, is_surjective_fn, is_bijection_fn,
    injective_fn_eq, surjective_fn_has_preimage, bijection_fn_is_injective,
    bijection_fn_is_surjective
from pair.base import Pair, pair_ext, pair_map, pair_map_first_eq,
    pair_map_second_eq, pair_new_first, pair_new_second

/// Componentwise pair mapping is injective when both component maps are injective.
theorem pair_map_injective_fn[A, B, C, D](f: A -> C, g: B -> D) {
    is_injective_fn(f) and is_injective_fn(g) implies
        is_injective_fn(pair_map[A, B, C, D](f, g))
} by {
    if is_injective_fn(f) and is_injective_fn(g) {
        forall(p: Pair[A, B], q: Pair[A, B]) {
            if pair_map[A, B, C, D](f, g, p) = pair_map[A, B, C, D](f, g, q) {
                pair_map[A, B, C, D](f, g, p).first =
                    pair_map[A, B, C, D](f, g, q).first
                pair_map_first_eq(f, g, p)
                pair_map[A, B, C, D](f, g, p).first = f(p.first)
                pair_map_first_eq(f, g, q)
                pair_map[A, B, C, D](f, g, q).first = f(q.first)
                f(p.first) = f(q.first)
                injective_fn_eq(f, p.first, q.first)
                p.first = q.first

                pair_map[A, B, C, D](f, g, p).second =
                    pair_map[A, B, C, D](f, g, q).second
                pair_map_second_eq(f, g, p)
                pair_map[A, B, C, D](f, g, p).second = g(p.second)
                pair_map_second_eq(f, g, q)
                pair_map[A, B, C, D](f, g, q).second = g(q.second)
                g(p.second) = g(q.second)
                injective_fn_eq(g, p.second, q.second)
                p.second = q.second

                pair_ext(p, q)
                p = q
            }
        }
        is_injective_fn(pair_map[A, B, C, D](f, g))
    }
}

/// Componentwise pair mapping is surjective when both component maps are surjective.
theorem pair_map_surjective_fn[A, B, C, D](f: A -> C, g: B -> D) {
    is_surjective_fn(f) and is_surjective_fn(g) implies
        is_surjective_fn(pair_map[A, B, C, D](f, g))
} by {
    if is_surjective_fn(f) and is_surjective_fn(g) {
        forall(q: Pair[C, D]) {
            surjective_fn_has_preimage(f, q.first)
            let x: A satisfy {
                f(x) = q.first
            }
            surjective_fn_has_preimage(g, q.second)
            let y: B satisfy {
                g(y) = q.second
            }
            let p = Pair.new(x, y)

            pair_map_first_eq(f, g, p)
            pair_map[A, B, C, D](f, g, p).first = f(p.first)
            pair_new_first(x, y)
            p.first = x
            pair_map[A, B, C, D](f, g, p).first = f(x)
            pair_map[A, B, C, D](f, g, p).first = q.first

            pair_map_second_eq(f, g, p)
            pair_map[A, B, C, D](f, g, p).second = g(p.second)
            pair_new_second(x, y)
            p.second = y
            pair_map[A, B, C, D](f, g, p).second = g(y)
            pair_map[A, B, C, D](f, g, p).second = q.second

            pair_ext(pair_map[A, B, C, D](f, g, p), q)
            pair_map[A, B, C, D](f, g, p) = q
            exists(preimage: Pair[A, B]) {
                preimage = p and pair_map[A, B, C, D](f, g, preimage) = q
            }
        }
        is_surjective_fn(pair_map[A, B, C, D](f, g))
    }
}

/// Componentwise pair mapping is bijective when both component maps are bijective.
theorem pair_map_bijection_fn[A, B, C, D](f: A -> C, g: B -> D) {
    is_bijection_fn(f) and is_bijection_fn(g) implies
        is_bijection_fn(pair_map[A, B, C, D](f, g))
} by {
    if is_bijection_fn(f) and is_bijection_fn(g) {
        bijection_fn_is_injective(f)
        is_injective_fn(f)
        bijection_fn_is_injective(g)
        is_injective_fn(g)
        pair_map_injective_fn(f, g)
        is_injective_fn(pair_map[A, B, C, D](f, g))

        bijection_fn_is_surjective(f)
        is_surjective_fn(f)
        bijection_fn_is_surjective(g)
        is_surjective_fn(g)
        pair_map_surjective_fn(f, g)
        is_surjective_fn(pair_map[A, B, C, D](f, g))

        is_injective_fn(pair_map[A, B, C, D](f, g)) and
            is_surjective_fn(pair_map[A, B, C, D](f, g))
        is_bijection_fn(pair_map[A, B, C, D](f, g))
    }
}
