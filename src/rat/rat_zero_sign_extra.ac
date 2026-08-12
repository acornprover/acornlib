from int import Int, zero_not_neg, zero_not_pos
from rat.rat_base import Rat, reduce, reduce_idempotent, reduce_zero_num,
    zero_num, from_int_zero, from_int_num, not_pos_and_neg, single_trichotomy,
    abs_non_pos, abs_non_neg, abs_zero_imp_zero
from rat.rat_props import pos_ne_zero

numerals Rat

/// Reducing a fraction with zero denominator uses the library's totalized value zero.
theorem reduce_zero_denominator(a: Int) {
    reduce(a, Int.0) = Rat.0
}

/// A rational with zero stored numerator is the zero rational.
theorem rat_num_zero_imp_zero(q: Rat) {
    q.num = Int.0 implies q = Rat.0
} by {
    if q.num = Int.0 {
        reduce_idempotent(q)
        reduce(q.num, q.denom) = q
        reduce_zero_num(q.denom)
        reduce(Int.0, q.denom) = Rat.0
        q = Rat.0
    }
}

/// Being the zero rational is equivalent to having zero stored numerator.
theorem rat_zero_iff_num_zero(q: Rat) {
    (q = Rat.0) = (q.num = Int.0)
} by {
    if q = Rat.0 {
        zero_num
        q.num = Int.0
    }
    if q.num = Int.0 {
        rat_num_zero_imp_zero(q)
    }
}

/// Being nonzero is equivalent to having nonzero stored numerator.
theorem rat_nonzero_iff_num_nonzero(q: Rat) {
    (q != Rat.0) = (q.num != Int.0)
} by {
    if q != Rat.0 {
        if q.num = Int.0 {
            rat_num_zero_imp_zero(q)
            false
        }
        q.num != Int.0
    }
    if q.num != Int.0 {
        if q = Rat.0 {
            zero_num
            q.num = Int.0
            false
        }
        q != Rat.0
    }
}

/// Zero is not positive.
theorem rat_zero_not_positive {
    not Rat.0.is_positive
} by {
    zero_num
    zero_not_pos
}

/// Zero is not negative.
theorem rat_zero_not_negative {
    not Rat.0.is_negative
} by {
    zero_num
    zero_not_neg
}

/// Positive rationals are nonzero.
theorem rat_positive_imp_nonzero(q: Rat) {
    q.is_positive implies q != Rat.0
} by {
    pos_ne_zero(q)
}

/// Negative rationals are nonzero.
theorem rat_negative_imp_nonzero(q: Rat) {
    q.is_negative implies q != Rat.0
} by {
    if q.is_negative {
        if q = Rat.0 {
            rat_zero_not_negative
            false
        }
        q != Rat.0
    }
}

/// Negative rationals are not positive.
theorem rat_negative_imp_not_positive(q: Rat) {
    q.is_negative implies not q.is_positive
} by {
    if q.is_negative {
        if q.is_positive {
            not_pos_and_neg(q)
            false
        }
        not q.is_positive
    }
}

/// A nonzero rational has one of the two signs.
theorem rat_nonzero_imp_positive_or_negative(q: Rat) {
    q != Rat.0 implies q.is_positive or q.is_negative
} by {
    if q != Rat.0 {
        single_trichotomy(q)
        if q = Rat.0 {
            false
        }
        q.is_positive or q.is_negative
    }
}

/// The absolute value of zero is zero.
theorem rat_abs_zero {
    Rat.0.abs = Rat.0
} by {
    rat_zero_not_negative
    abs_non_neg(Rat.0)
}

/// A rational has zero absolute value exactly when it is zero.
theorem rat_abs_zero_iff_zero(q: Rat) {
    (q.abs = Rat.0) = (q = Rat.0)
} by {
    if q.abs = Rat.0 {
        abs_zero_imp_zero(q)
    }
    if q = Rat.0 {
        rat_abs_zero
        q.abs = Rat.0
    }
}

/// Positive rationals are fixed by absolute value.
theorem rat_abs_positive_self(q: Rat) {
    q.is_positive implies q.abs = q
} by {
    if q.is_positive {
        not_pos_and_neg(q)
        abs_non_neg(q)
    }
}

/// Negative rationals have absolute value equal to their negation.
theorem rat_abs_negative_neg(q: Rat) {
    q.is_negative implies q.abs = -q
} by {
    if q.is_negative {
        rat_negative_imp_not_positive(q)
        abs_non_pos(q)
    }
}

/// An embedded integer is zero exactly when the integer is zero.
theorem from_int_zero_iff(n: Int) {
    (Rat.from_int(n) = Rat.0) = (n = Int.0)
} by {
    if Rat.from_int(n) = Rat.0 {
        from_int_num(n)
        zero_num
        Rat.from_int(n).num = n
        Rat.0.num = Int.0
        n = Int.0
    }
    if n = Int.0 {
        from_int_zero
    }
}

/// An embedded integer is nonzero exactly when the integer is nonzero.
theorem from_int_nonzero_iff(n: Int) {
    (Rat.from_int(n) != Rat.0) = (n != Int.0)
} by {
    if Rat.from_int(n) != Rat.0 {
        if n = Int.0 {
            from_int_zero
            false
        }
        n != Int.0
    }
    if n != Int.0 {
        if Rat.from_int(n) = Rat.0 {
            from_int_zero_iff(n)
            n = Int.0
            false
        }
        Rat.from_int(n) != Rat.0
    }
}

/// Positivity is preserved exactly by the integer embedding.
theorem from_int_positive_iff(n: Int) {
    Rat.from_int(n).is_positive = n.is_positive
} by {
    from_int_num(n)
}

/// Negativity is preserved exactly by the integer embedding.
theorem from_int_negative_iff(n: Int) {
    Rat.from_int(n).is_negative = n.is_negative
} by {
    from_int_num(n)
}
