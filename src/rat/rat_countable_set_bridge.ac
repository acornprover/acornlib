from nat import Nat
from data.basic.set import Set, set_image, all_sets_subset_universal,
    set_image_universal_of_surjective
from data.cardinal.countable import is_countable, countable_has_sequence_of_nonempty,
    subset_of_countable_is_countable, image_of_countable_is_countable,
    nat_universal_is_countable
from rat.rat_base import Rat
from rat.rat_countable import rat_zigzag, rat_zigzag_surjective,
    rat_zigzag_has_preimage

/// The universal set of rational numbers is countable.
theorem rat_universal_set_is_countable {
    is_countable[Rat](Set[Rat].universal_set)
} by {
    is_countable[Rat](Set[Rat].universal_set) =
        (Set[Rat].universal_set = Set[Rat].empty_set or exists(f: Nat -> Rat) {
            forall(x: Rat) {
                Set[Rat].universal_set.contains(x) implies exists(n: Nat) { f(n) = x }
            }
        })
    forall(x: Rat) {
        if Set[Rat].universal_set.contains(x) {
            rat_zigzag_has_preimage(x)
            exists(n: Nat) { rat_zigzag(n) = x }
        }
    }
    exists(f: Nat -> Rat) {
        forall(x: Rat) {
            Set[Rat].universal_set.contains(x) implies exists(n: Nat) { f(n) = x }
        }
    }
    is_countable[Rat](Set[Rat].universal_set)
}

/// Every set of rational numbers is countable.
theorem rat_subset_is_countable(s: Set[Rat]) {
    is_countable[Rat](s)
} by {
    all_sets_subset_universal(s)
    rat_universal_set_is_countable
    subset_of_countable_is_countable(s, Set[Rat].universal_set)
}

/// The range of the rational zig-zag enumeration is the universal rational set.
theorem rat_range_zigzag_is_universal {
    set_image(Set[Nat].universal_set, rat_zigzag) = Set[Rat].universal_set
} by {
    rat_zigzag_surjective
    set_image_universal_of_surjective(rat_zigzag)
}

/// The image of the natural-number embedding into the rationals is countable.
theorem rat_from_nat_image_is_countable {
    is_countable[Rat](set_image(Set[Nat].universal_set, Rat.from_nat))
} by {
    nat_universal_is_countable
    image_of_countable_is_countable(Set[Nat].universal_set, Rat.from_nat)
}

/// Every nonempty set of rational numbers has a sequence covering its elements.
theorem rat_countable_has_sequence(s: Set[Rat]) {
    s != Set[Rat].empty_set implies exists(f: Nat -> Rat) {
        forall(x: Rat) { s.contains(x) implies exists(n: Nat) { f(n) = x } }
    }
} by {
    if s != Set[Rat].empty_set {
        rat_subset_is_countable(s)
        countable_has_sequence_of_nonempty(s)
    }
}
