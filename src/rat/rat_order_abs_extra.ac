from int import Int
from rat.rat_base import Rat, reduce, cross_equals, denom_nonzero,
    denom_positive, reduce_zero_num, reduce_pos_pos, cross_eq_imp_eq,
    cross_eq_imp_reduce_eq, reduce_eq_imp_cross_eq, pos_imp_zero_lt, zero_lt_imp_pos, neg_imp_lt_zero,
    lt_zero_imp_neg, not_lt_self, trichotomy, lte_trans, lt_trans, lte_lt_trans,
    lt_lte_trans, zero_lte_abs, lte_abs, neg_abs
from rat.rat_props import reduce_abs, reduce_nonneg, reduce_neg_num, reduce_neg_denom,
    cross_mul_lt, cross_mul_lte, abs_nonneg
numerals Rat

/// A rational denominator is positive exactly in the integer positivity sense.
theorem rat_denom_positive_iff(r: Rat) {
    r.denom.is_positive = (Int.0 < r.denom)
} by {
    denom_positive(r)
    r.denom.is_positive
    r.denom > Int.0
}

/// A positive rational denominator is nonzero, as a reusable bridge.
theorem rat_denom_positive_imp_nonzero(r: Rat) {
    r.denom.is_positive implies r.denom != Int.0
} by {
    if r.denom.is_positive {
        denom_nonzero(r)
    }
}

/// Cross equality is available for the stored numerator and denominator of two
/// equal rationals.
theorem rat_eq_imp_cross_equals(r: Rat, s: Rat) {
    r = s implies cross_equals(r.num, r.denom, s.num, s.denom)
} by {
    if r = s {
        denom_nonzero(r)
        denom_nonzero(s)
        r.denom != Int.0
        s.denom != Int.0
        r.num = s.num
        r.denom = s.denom
        r.num * s.denom = r.num * r.denom
        s.num * r.denom = r.num * r.denom
        r.num * s.denom = s.num * r.denom
        cross_equals(r.num, r.denom, s.num, s.denom) =
            (r.denom != Int.0 and s.denom != Int.0 and r.num * s.denom = s.num * r.denom)
        cross_equals(r.num, r.denom, s.num, s.denom)
    }
}

/// Equality of rationals is equivalent to the cross-multiplication criterion on
/// their stored reduced representatives.
theorem rat_eq_iff_cross_equals(r: Rat, s: Rat) {
    (r = s) = cross_equals(r.num, r.denom, s.num, s.denom)
} by {
    if r = s {
        rat_eq_imp_cross_equals(r, s)
    }
    if cross_equals(r.num, r.denom, s.num, s.denom) {
        cross_eq_imp_eq(r, s)
    }
}

/// A nonzero-denominator reduction exposes cross equality to the unreduced
/// numerator and denominator.
theorem reduce_cross_equals(a: Int, b: Int) {
    b != Int.0 implies cross_equals(reduce(a, b).num, reduce(a, b).denom, a, b)
} by {
    if b != Int.0 {
        cross_equals(reduce(a, b).num, reduce(a, b).denom, a, b) or Int.0 = b
        cross_equals(reduce(a, b).num, reduce(a, b).denom, a, b)
    }
}

/// Reducing an already represented rational by its stored denominator returns it.
theorem reduce_num_denom(r: Rat) {
    reduce(r.num, r.denom) = r
} by {
    denom_nonzero(r)
    reduce_cross_equals(r.num, r.denom)
    cross_equals(reduce(r.num, r.denom).num, reduce(r.num, r.denom).denom, r.num, r.denom)
    cross_eq_imp_eq(reduce(r.num, r.denom), r)
}

/// Equality of two nonzero-denominator reductions gives the usual cross product.
theorem reduce_eq_imp_cross_product(a: Int, b: Int, c: Int, d: Int) {
    b != Int.0 and d != Int.0 and reduce(a, b) = reduce(c, d)
        implies a * d = c * b
} by {
    if b != Int.0 and d != Int.0 and reduce(a, b) = reduce(c, d) {
        reduce_eq_imp_cross_eq(a, b, c, d)
        cross_equals(a, b, c, d)
        a * d = c * b
    }
}

/// Cross product equality over nonzero denominators gives equality of reduced
/// rationals.
theorem cross_product_imp_reduce_eq(a: Int, b: Int, c: Int, d: Int) {
    b != Int.0 and d != Int.0 and a * d = c * b
        implies reduce(a, b) = reduce(c, d)
} by {
    if b != Int.0 and d != Int.0 and a * d = c * b {
        cross_equals(a, b, c, d)
        cross_eq_imp_reduce_eq(a, b, c, d)
    }
}

/// Cross product equality is equivalent to equality of reductions when both
/// denominators are nonzero.
theorem reduce_eq_iff_cross_product(a: Int, b: Int, c: Int, d: Int) {
    b != Int.0 and d != Int.0 implies
        (reduce(a, b) = reduce(c, d)) = (a * d = c * b)
} by {
    if b != Int.0 and d != Int.0 {
        if reduce(a, b) = reduce(c, d) {
            reduce_eq_imp_cross_product(a, b, c, d)
            a * d = c * b
        }
        if a * d = c * b {
            cross_product_imp_reduce_eq(a, b, c, d)
            reduce(a, b) = reduce(c, d)
        }
        (reduce(a, b) = reduce(c, d)) = (a * d = c * b)
    }
}

/// Reducing a fraction with zero numerator gives zero.
theorem reduce_zero_num_eq_zero(b: Int) {
    reduce(Int.0, b) = Rat.0
} by {
    reduce_zero_num(b)
}

/// Reducing a positive-over-positive integer fraction gives a positive rational.
theorem reduce_positive_of_positive(a: Int, b: Int) {
    a.is_positive and b.is_positive implies reduce(a, b).is_positive
} by {
    if a.is_positive and b.is_positive {
        reduce_pos_pos(a, b)
    }
}

/// Reducing a nonnegative-over-nonnegative integer fraction gives a nonnegative
/// rational in the sign-predicate sense.
theorem reduce_nonnegative_of_nonnegative(a: Int, b: Int) {
    not a.is_negative and not b.is_negative implies not reduce(a, b).is_negative
} by {
    if not a.is_negative and not b.is_negative {
        reduce_nonneg(a, b)
    }
}

/// Absolute value may be pushed through both numerator and denominator of a
/// reduction.
theorem reduce_abs_num_denom(a: Int, b: Int) {
    reduce(a.abs, b.abs) = reduce(a, b).abs
} by {
    reduce_abs(a, b)
}

/// Negating both numerator and denominator leaves a reduced fraction unchanged.
theorem reduce_neg_neg(a: Int, b: Int) {
    reduce(-a, -b) = reduce(a, b)
} by {
    reduce_neg_num(a, -b)
    reduce(-a, -b) = -reduce(a, -b)
    reduce_neg_denom(a, b)
    reduce(a, -b) = -reduce(a, b)
    -reduce(a, -b) = --reduce(a, b)
    --reduce(a, b) = reduce(a, b)
}

/// Positivity of a rational is equivalent to being strictly greater than zero.
theorem rat_positive_iff_zero_lt(q: Rat) {
    q.is_positive = (Rat.0 < q)
} by {
    if q.is_positive {
        pos_imp_zero_lt(q)
        Rat.0 < q
    }
    if Rat.0 < q {
        zero_lt_imp_pos(q)
        q.is_positive
    }
}

/// Negativity of a rational is equivalent to being strictly less than zero.
theorem rat_negative_iff_lt_zero(q: Rat) {
    q.is_negative = (q < Rat.0)
} by {
    if q.is_negative {
        neg_imp_lt_zero(q)
        q < Rat.0
    }
    if q < Rat.0 {
        lt_zero_imp_neg(q)
        q.is_negative
    }
}

/// Rational strict order is irreflexive, with a descriptive name for clients.
theorem rat_lt_irrefl(q: Rat) {
    not q < q
} by {
    not_lt_self(q)
}

/// Rational trichotomy as a compact client-facing endpoint.
theorem rat_trichotomy_cases(a: Rat, b: Rat) {
    a < b or a = b or b < a
} by {
    trichotomy(a, b)
}

/// The absolute value of a rational is nonnegative in order form.
theorem rat_abs_nonnegative(q: Rat) {
    Rat.0 <= q.abs
} by {
    zero_lte_abs(q)
}

/// Every rational is bounded above by its absolute value.
theorem rat_lte_abs_self(q: Rat) {
    q <= q.abs
} by {
    lte_abs(q)
}

/// Negating a rational does not change its absolute value.
theorem rat_abs_neg(q: Rat) {
    (-q).abs = q.abs
} by {
    neg_abs(q)
}

/// Absolute value as a sign predicate is never negative.
theorem rat_abs_not_negative(q: Rat) {
    not q.abs.is_negative
} by {
    abs_nonneg(q)
}

/// Cross multiplication wrapper for strict inequalities between fractions.
theorem rat_cross_mul_lt_fraction(a: Rat, b: Rat, c: Rat, d: Rat) {
    b.is_positive and d.is_positive and a * d < c * b implies a / b < c / d
} by {
    if b.is_positive and d.is_positive and a * d < c * b {
        cross_mul_lt(a, b, c, d)
    }
}

/// Cross multiplication wrapper for non-strict inequalities between fractions.
theorem rat_cross_mul_lte_fraction(a: Rat, b: Rat, c: Rat, d: Rat) {
    b.is_positive and d.is_positive and a * d <= c * b implies a / b <= c / d
} by {
    if b.is_positive and d.is_positive and a * d <= c * b {
        cross_mul_lte(a, b, c, d)
    }
}

/// Transitivity wrapper for rational non-strict order.
theorem rat_lte_transitive(a: Rat, b: Rat, c: Rat) {
    a <= b and b <= c implies a <= c
} by {
    if a <= b and b <= c {
        lte_trans(a, b, c)
    }
}

/// Transitivity wrapper for rational strict order.
theorem rat_lt_transitive(a: Rat, b: Rat, c: Rat) {
    a < b and b < c implies a < c
} by {
    if a < b and b < c {
        lt_trans(a, b, c)
    }
}

/// Mixed transitivity wrapper: `<=` followed by `<`.
theorem rat_lte_lt_transitive(a: Rat, b: Rat, c: Rat) {
    a <= b and b < c implies a < c
} by {
    if a <= b and b < c {
        lte_lt_trans(a, b, c)
    }
}

/// Mixed transitivity wrapper: `<` followed by `<=`.
theorem rat_lt_lte_transitive(a: Rat, b: Rat, c: Rat) {
    a < b and b <= c implies a < c
} by {
    if a < b and b <= c {
        lt_lte_trans(a, b, c)
    }
}
