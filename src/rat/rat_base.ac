from algebra.add import Add
from algebra.inverse import Inverse
from lte import LTE
from algebra.mul import Mul
from nat import Nat, alt_suc_ne_zero
from algebra.neg import Neg
from int import Int, unit_sign, abs, abs_from_nat, abs_zero

/// True if a fraction a/b is in reduced form (lowest terms).
/// The denominator must be positive and gcd(a,b) must be 1.
define is_reduced(a: Int, b: Int) -> Bool {
    b > Int.0 and a.gcd(b) = Int.1
}

theorem denom_one_is_reduced(a: Int) {
    is_reduced(a, Int.1)
} by {
    Int.1 > Int.0
}

/// Rational numbers represented as fractions in reduced form.
/// The constraint ensures the fraction is always in lowest terms with positive denominator.
structure Rat {
    /// The numerator of the rational number.
    num: Int
    /// The denominator of the rational number (always positive).
    denom: Int
} constraint {
    is_reduced(num, denom)
}

let Option.some(rat_zero) = Rat.new(Int.0, Int.1)

from algebra.zero import Zero

instance Rat: Zero {
    let 0: Rat = rat_zero
}

theorem zero_num {
    Rat.0.num = Int.0
}

theorem zero_denom {
    Rat.0.denom = Int.1
}

theorem denom_nonzero(r: Rat) {
    r.denom != Int.0
} by {
    r.denom > Int.0
}

theorem denom_positive(r: Rat) {
    r.denom.is_positive
} by {
    r.denom > Int.0
}

let rat_from_int(n: Int) -> r: Rat satisfy {
    Option.some(r) = Rat.new(n, Int.1)
}

theorem constraint_yields_new(a: Int, b: Int) {
    Rat.constraint(a, b) implies exists(r: Rat) {
        Option.some(r) = Rat.new(a, b)
    }
} by {
    if Rat.constraint(a, b) {
        Rat.constraint(a, b)
    }
}

attributes Rat {
    /// Converts an integer to a rational number.
    let from_int = rat_from_int
}

theorem from_int_zero {
    Rat.from_int(Int.0) = Rat.0
}

theorem from_int_num(n: Int) {
    Rat.from_int(n).num = n
}

theorem from_int_denom(n: Int) {
    Rat.from_int(n).denom = Int.1
}

/// True if two fractions a/b and c/d are equal using cross multiplication.
/// Requires non-zero denominators to avoid the 0/0 edge case.
define cross_equals(a: Int, b: Int, c: Int, d: Int) -> Bool {
    b != Int.0 and d != Int.0 and a * d = c * b
}

theorem cross_equals_trans(a: Int, b: Int, c: Int, d: Int, e: Int, f: Int) {
    cross_equals(a, b, c, d) and cross_equals(c, d, e, f) implies cross_equals(a, b, e, f)
} by {
    b * (c * f) = b * c * f
    d * (a * f) = d * a * f
    d * a = a * d
    c * b = b * c
    d * (a * f) = a * f * d
    b * (e * d) = b * e * d
    a * f * d = b * e * d
}

/// Reduces a fraction a/b to its simplest form as a Rat.
/// Yields 0 if b = 0 (treating division by zero as zero).
let reduce(a: Int, b: Int) -> r: Rat satisfy {
    if b = Int.0 {
        r = Rat.0
    } else {
        cross_equals(r.num, r.denom, a, b)
    }
} by {
    if b = Int.0 {
    } else {
        // Factor out the gcd
        let a1: Int satisfy {
            a1 * a.gcd(b) = a
        }
        let b1: Int satisfy {
            b1 * a.gcd(b) = b
        }
        a * b1 = b * a1
        a1.gcd(b1) = Int.1

        // To reduce we also need to swap signs if the denominator is negative
        let a2 = a1 * unit_sign(b1)
        let b2 = b1 * unit_sign(b1)
        unit_sign(b1) * b1 = Int.from_nat(abs(b1))
        unit_sign(b1) * Int.from_nat(abs(b1)) = b1
        b1 * unit_sign(b1) = unit_sign(b1) * b1
        -b != b or Int.0 = b
        -b1 != b1 or Int.0 = b1
        --b1 = b1
        --b1 * a.gcd(b) = -(-b1 * a.gcd(b))
        Int.0 - b1 = -b1
        b1 - Int.0 = b1
        unit_sign(b1) * b1 - unit_sign(b1) * b2 = unit_sign(b1) * (b1 - b2)
        b2 != Int.0
        (a1 * unit_sign(b1)) * b1 = (b1 * unit_sign(b1)) * a1
        cross_equals(a2, b2, a1, b1)
        a2.gcd(b2) = Int.1 * Int.from_nat(abs(unit_sign(b1)))
        Int.from_nat(abs(unit_sign(b1))) = Int.1
        b2 > Int.0
        Int.1 * Int.from_nat(abs(unit_sign(b1))) = Int.from_nat(abs(unit_sign(b1)))
        a2.gcd(b2) = Int.1
        is_reduced(a2, b2)
        Rat.constraint(a2, b2)
        exists(r0: Rat) {
            Option.some(r0) = Rat.new(a2, b2)
        }

        let Option.some(r) = Rat.new(a2, b2)
        r.denom = b2
        cross_equals(a2, b2, a, b)
        cross_equals(r.num, r.denom, a, b)
    }
}

theorem cross_eq_imp_eq(r1: Rat, r2: Rat) {
    cross_equals(r1.num, r1.denom, r2.num, r2.denom) implies r1 = r2
} by {
    // cross_equals gives us r2.num * r1.denom = r1.num * r2.denom
    not cross_equals(r1.num, r1.denom, r2.num, r2.denom) or r2.num * r1.denom = r1.num * r2.denom

    // The r1 fields divide into the r2 ones
    // is_reduced(r1.num, r1.denom) from Rat definition
    is_reduced(r1.num, r1.denom)
    r1.num.gcd(r1.denom) = Int.1
    r1.denom.gcd(r1.num) = r1.num.gcd(r1.denom)
    r1.denom.gcd(r1.num) = Int.1
    // r1.denom divides r1.num * r2.denom, so by Euclid's lemma it divides r2.denom
    r1.denom.divides(r1.num * r2.denom)
    r1.denom.divides(r2.denom)

    // The r2 fields divide into the r1 ones
    r2.denom.divides(r2.num * r1.denom)
    abs(r2.denom).divides(abs(r1.denom))

    // Prove int equality from nat equality
    r1.denom = r2.denom

    // Divide out the denominators
    r1.denom.is_positive
    r2.num * r1.denom != r1.num * r1.denom or not r1.denom.is_positive or r2.num = r1.num
    Rat.new(r1.num, r1.denom) = Option.some(r1)
    Rat.new(r2.num, r2.denom) = Option.some(r2)
}

attributes Rat {
    /// The rational two.
    let 2: Rat = Rat.from_int(Int.2)
    /// The rational three.
    let 3: Rat = Rat.from_int(Int.3)
    /// The rational four.
    let 4: Rat = Rat.from_int(Int.4)
    /// The rational five.
    let 5: Rat = Rat.from_int(Int.5)
    /// The rational six.
    let 6: Rat = Rat.from_int(Int.6)
    /// The rational seven.
    let 7: Rat = Rat.from_int(Int.7)
    /// The rational eight.
    let 8: Rat = Rat.from_int(Int.8)
    /// The rational nine.
    let 9: Rat = Rat.from_int(Int.9)
    /// The rational ten.
    let 10: Rat = Rat.from_int(Int.10)

    /// True if the rational is positive.
    define is_positive(self) -> Bool {
        self.num.is_positive
    }

    /// True if the rational is negative.
    define is_negative(self) -> Bool {
        self.num.is_negative
    }
}

from algebra.one import One

instance Rat: One {
    let 1: Rat = Rat.from_int(Int.1)
}

let neg(a: Rat) -> b: Rat satisfy {
    Rat.new(-a.num, a.denom) = Option.some(b)
} by {
    is_reduced(a.num, a.denom)
    is_reduced(-a.num, a.denom)
}

/// The negation of a rational number.
instance Rat: Neg {
    let neg = neg
}

/// The sum of two rational numbers.
instance Rat: Add {
    define add(self, other: Rat) -> Rat {
        reduce(self.num * other.denom + other.num * self.denom,
               self.denom * other.denom)
    }
}

/// The product of two rational numbers.
instance Rat: Mul {
    define mul(self, other: Rat) -> Rat {
        reduce(self.num * other.num, self.denom * other.denom)
    }
}

/// The inverse of a rational number (1/x).
/// The inverse of zero is defined to be zero.
instance Rat: Inverse {
    define inverse(self) -> Rat {
        reduce(self.denom, self.num)
    }
}

attributes Rat {
    /// The quotient of two rational numbers.
    /// Division by zero is defined to yield zero.
    define div(self, other: Rat) -> Rat {
        self * other.inverse
    }

    /// The rational formed by appending a digit to this rational in base 10.
    define read(self, other: Rat) -> Rat { Rat.10 * self + other }
}

theorem reduce_idempotent(r: Rat) {
    reduce(r.num, r.denom) = r
}

theorem neg_is_reduced(r: Rat) {
    is_reduced(-r.num, r.denom)
} by {
    r.denom > Int.0
    is_reduced(r.num, r.denom)
    r.num.gcd(r.denom) = Int.1
    abs(-r.num) = abs(r.num)
    (-r.num).gcd(r.denom) = Int.1
}

theorem neg_num(r: Rat) {
    r.neg.num = -r.num
}

theorem neg_denom(r: Rat) {
    r.neg.denom = r.denom
}

theorem add_zero_right(a: Rat) {
    a + Rat.0 = a
}

theorem add_zero_left(a: Rat) {
    Rat.0 + a = a
}

theorem add_comm(a: Rat, b: Rat) {
    a + b = b + a
}

theorem from_int_cancel(a: Int, b: Int) {
    Rat.from_int(a) = Rat.from_int(b) implies a = b
}

theorem mul_comm(a: Rat, b: Rat) {
    a * b = b * a
}

theorem rat_neg_one {
    -Rat.1 = Rat.from_int(-Int.1)
} by {
    (-Rat.1).num = -Rat.1.num
    (-Rat.1).denom = Rat.1.denom
    Rat.from_int(Int.1).num = Int.1
    Rat.from_int(Int.1).denom = Int.1
    Rat.from_int(-Int.1).num = -Int.1
    Rat.from_int(-Int.1).denom = Int.1
    reduce((-Rat.1).num, (-Rat.1).denom) = -Rat.1
    reduce(Rat.from_int(-Int.1).num, Rat.from_int(-Int.1).denom) = Rat.from_int(-Int.1)
}

theorem mul_neg_one_right(r: Rat) {
    r * -Rat.1 = -r
} by {
    reduce(r.num * (-Rat.1).num, r.denom * (-Rat.1).denom) = r * -Rat.1
    r.num * -Rat.1.num = -(r.num * Rat.1.num)
    (-r).num = -r.num
    reduce((-r).num, (-r).denom) = -r
    r.denom * (-Rat.1).denom = (-Rat.1).denom * r.denom
    (-Rat.1).denom = Rat.1.denom
    (-r).denom = r.denom
    Rat.from_int(Int.1).denom = Int.1
    Rat.from_int(-Int.1).num = -Int.1
    Rat.from_int(Int.1).num = Int.1
    Int.1 * r.denom = r.denom
    r.num * Int.1 = r.num
}

theorem mul_neg_one_left(r: Rat) {
    -Rat.1 * r = -r
}

theorem mul_one_right(r: Rat) {
    r * Rat.1 = r
} by {
    Rat.from_int(Int.1).num = Int.1
    Rat.from_int(Int.1).denom = Int.1
    r.num * Int.1 = r.num
    Int.1 * r.denom = r.denom
    reduce(r.num, r.denom) = r
}

theorem mul_one_left(r: Rat) {
    Rat.1 * r = r
}

theorem mul_int_eq_int_mul(a: Int, b: Int) {
    Rat.from_int(a) * Rat.from_int(b) = Rat.from_int(a * b)
}

theorem unreduce_right(a: Int, b: Int) {
    b != Int.0 implies exists(d: Int) {
        reduce(a, b).num * d = a and reduce(a, b).denom * d = b
    }
} by {
    let r = reduce(a, b)
    cross_equals(reduce(a, b).num, reduce(a, b).denom, a, b) or Int.0 = b
    not cross_equals(r.num, r.denom, a, b) or a * r.denom = r.num * b
    r.num * b = a * r.denom
    is_reduced(r.num, r.denom)
    not is_reduced(r.num, r.denom) or r.num.gcd(r.denom) = Int.1
    r.denom.gcd(r.num) = r.num.gcd(r.denom)
    a * r.denom != r.num * b or r.denom.divides(r.num * b)
    not r.denom.divides(r.num * b) or r.denom.gcd(r.num) != Int.1 or r.denom.divides(b)
    r.denom.divides(b)
    let d: Int satisfy {
        d * r.denom = b
    }
    r.denom * d = b
    r.num * d * b = a * b
}

theorem unreduce_left(a: Int, b: Int) {
    b != Int.0 implies exists(d: Int) {
        d * reduce(a, b).num = a and d * reduce(a, b).denom = b
    }
} by {
    let d: Int satisfy {
        reduce(a, b).num * d = a and reduce(a, b).denom * d = b
    }
}

theorem mul_int_right(r: Rat, n: Int) {
    r * Rat.from_int(n) = reduce(r.num * n, r.denom)
}

theorem mul_int_left(n: Int, r: Rat) {
    Rat.from_int(n) * r = reduce(n * r.num, r.denom)
}

theorem cross_eq_imp_reduce_eq(a: Int, b: Int, c: Int, d: Int) {
    cross_equals(a, b, c, d) implies reduce(a, b) = reduce(c, d)
} by {
    // cross_equals implies b != 0 and d != 0
    not cross_equals(a, b, c, d) or b != Int.0
    not cross_equals(a, b, c, d) or d != Int.0
    let rab = reduce(a, b)
    // reduce definition gives cross_equals
    cross_equals(reduce(a, b).num, reduce(a, b).denom, a, b) or Int.0 = b
    cross_equals(rab.num, rab.denom, a, b)
    let rcd = reduce(c, d)
    cross_equals(reduce(c, d).num, reduce(c, d).denom, c, d) or Int.0 = d
    cross_equals(rcd.num, rcd.denom, c, d)
    // Use cross_equals_trans: (a,b) ~ (c,d) and (c,d) ~ rcd => (a,b) ~ rcd
    not cross_equals(a, b, c, d) or not cross_equals(c, d, rcd.num, rcd.denom) or cross_equals(a, b, rcd.num, rcd.denom)
    // Use cross_equals_trans: rab ~ (a,b) and (a,b) ~ rcd => rab ~ rcd
    not cross_equals(rab.num, rab.denom, a, b) or not cross_equals(a, b, rcd.num, rcd.denom) or cross_equals(rab.num, rab.denom, rcd.num, rcd.denom)
    // cross_eq_imp_eq gives rab = rcd
    rcd.denom != Int.0
    rab = rcd
}

theorem reduce_eq_imp_cross_eq(a: Int, b: Int, c: Int, d: Int) {
    b != Int.0 and d != Int.0 and reduce(a, b) = reduce(c, d)
        implies cross_equals(a, b, c, d)
} by {
    cross_equals(a, b, reduce(a, b).num, reduce(c, d).denom)
}

theorem mul_reduced_int_right(a: Int, b: Int, c: Int) {
    reduce(a, b) * Rat.from_int(c) = reduce(a * c, b)
} by {
    if b = Int.0 {
        Int.0 != b or reduce(a, b) = Rat.0
        Int.0 != b or reduce(a * c, b) = Rat.0
        Int.0 * c = Int.0
        reduce(a, b) * Rat.from_int(c) = reduce(a * c, b)
    } else {
        let r = reduce(a, b)
        r.num * b = r.denom * a
        r.num * c * b = r.denom * (a * c)
        reduce(a, b) * Rat.from_int(c) = reduce(a * c, b)
    }
}

theorem mul_reduced_int_left(a: Int, b: Int, c: Int) {
    Rat.from_int(c) * reduce(a, b) = reduce(c * a, b)
}

theorem mul_int_right_cancel(r1: Rat, r2: Rat, n: Int) {
    n != Int.0 and r1 * Rat.from_int(n) = r2 * Rat.from_int(n) implies r1 = r2
} by {
    reduce(r1.num * n, r1.denom) = reduce(r2.num * n, r2.denom)
    r1.num * n * r2.denom = r1.denom * r2.num * n
    r1.num * r2.denom = r1.denom * r2.num
}

theorem mul_int_left_cancel(r1: Rat, r2: Rat, n: Int) {
    n != Int.0 and Rat.from_int(n) * r1 = Rat.from_int(n) * r2 implies r1 = r2
}

theorem mul_reduced_nondegen(a: Int, b: Int, c: Int, d: Int) {
    b != Int.0 and d != Int.0 implies
        reduce(a, b) * reduce(c, d) = reduce(a * c, b * d)
} by {
    let e: Int satisfy {
        reduce(a, b).num * e = a and reduce(a, b).denom * e = b
    }
    let f: Int satisfy {
        reduce(c, d).num * f = c and reduce(c, d).denom * f = d
    }

    // Simplify numerator
    let nums = reduce(a, b).num * reduce(c, d).num
    e * (f * reduce(c, d).num) = e * f * reduce(c, d).num
    reduce(a, b).num * (e * c) = reduce(a, b).num * e * c
    reduce(a, b).num * (reduce(c, d).num * (e * f)) = reduce(a, b).num * reduce(c, d).num * (e * f)
    f * reduce(c, d).num = reduce(c, d).num * f
    reduce(c, d).num * (e * f) = e * f * reduce(c, d).num
    (reduce(a, b).num * reduce(c, d).num) * (e * f) = a * c

    // Simplify denominator
    let denoms: Int = reduce(a, b).denom * reduce(c, d).denom
    reduce(a, b).denom * e * (reduce(c, d).denom * f) = reduce(a, b).denom * e * reduce(c, d).denom * f
    reduce(c, d).denom * (reduce(a, b).denom * e) = reduce(c, d).denom * reduce(a, b).denom * e
    reduce(c, d).denom * (reduce(a, b).denom * e) = reduce(a, b).denom * e * reduce(c, d).denom
    reduce(c, d).denom * reduce(a, b).denom = reduce(a, b).denom * reduce(c, d).denom
    (reduce(a, b).denom * reduce(c, d).denom) * e * f = b * d
    denoms * (e * f) = b * d

    // Combining
    denoms * (nums * (e * f)) = denoms * nums * (e * f)
    nums * (denoms * (e * f)) = nums * denoms * (e * f)
    denoms * nums = nums * denoms
    nums * (b * d) = denoms * (a * c)
    denoms != Int.0
    cross_equals(reduce(a, b).num * reduce(c, d).num,
                 reduce(a, b).denom * reduce(c, d).denom,
                 a * c, b * d)
}

theorem reduce_zero_num(a: Int) {
    reduce(Int.0, a) = Rat.0
} by {
    if a = Int.0 {
    } else {
        Int.0 * Int.1 = Int.0
        Int.0 * a = Int.0
        Int.0 * Int.1 != Int.0 * a or cross_equals(Int.0, Int.1, Int.0, a) or Int.1 = Int.0 or Int.0 = a
        not cross_equals(Int.0, Int.1, Int.0, a) or reduce(Int.0, Int.1) = reduce(Int.0, a)
        reduce(Int.0, a) = reduce(Int.0, Int.1)
    }
}

theorem mul_zero_right(r: Rat) {
    r * Rat.0 = Rat.0
}

theorem mul_reduced_degen(a: Int, b: Int, c: Int) {
    reduce(a, b) * reduce(c, Int.0) = reduce(a * c, b * Int.0)
}

theorem mul_reduced(a: Int, b: Int, c: Int, d: Int) {
    reduce(a, b) * reduce(c, d) = reduce(a * c, b * d)
} by {
    if b = Int.0 or d = Int.0 {
        // Degen cases
        if b = Int.0 {
            reduce(c, d) * Rat.0 = Rat.0 * reduce(c, d)
            Int.0 != b or reduce(a, b) = Rat.0
            reduce(c, d) * Rat.0 = Rat.0
            Int.0 * d = Int.0
            reduce(a * c, Int.0) = Rat.0
            reduce(a, b) * reduce(c, d) = reduce(a * c, b * d)
        } else {
        }
    } else {
        // Non-degen case
    }
}

theorem mul_assoc(a: Rat, b: Rat, c: Rat) {
    (a * b) * c = a * (b * c)
} by {
    reduce(a.num * b.num * c.num, a.denom * b.denom * c.denom) = reduce(a.num * b.num, a.denom * b.denom) * reduce(c.num, c.denom)
    reduce(a.num * b.num, a.denom * b.denom) = a * b
    a.denom * b.denom * c.denom = a.denom * (b.denom * c.denom)
    a.num * b.num * c.num = a.num * (b.num * c.num)
    reduce(c.num, c.denom) = c
    a * (b * c) = reduce(a.num * (b.num * c.num), a.denom * (b.denom * c.denom))
}

theorem add_int_eq_int_add(a: Int, b: Int) {
    Rat.from_int(a) + Rat.from_int(b) = Rat.from_int(a + b)
}

theorem reduce_cancels_right(a: Int, b: Int, c: Int) {
    c != Int.0 implies reduce(a, b) = reduce(a * c, b * c)
} by {
    if b = Int.0 {
    } else {
        reduce(a, b) = reduce(a * c, b * c)
    }
}

theorem reduce_cancels_left(a: Int, b: Int, c: Int) {
    c != Int.0 implies reduce(a, b) = reduce(c * a, c * b)
}

theorem add_reduce_right(r: Rat, a: Int, b: Int) {
    b != Int.0 implies r + reduce(a, b) = reduce(r.num * b + a * r.denom, r.denom * b)
} by {
    let c: Int satisfy {
        reduce(a, b).num * c = a and reduce(a, b).denom * c = b
    }
    forall(x0: Int, x1: Int) { reduce(x0 * c, x1 * c) = reduce(x0, x1) }
    r.denom * (reduce(a, b).denom * c) = r.denom * reduce(a, b).denom * c
    r.denom * (reduce(a, b).num * c) = r.denom * reduce(a, b).num * c
    r.num * (reduce(a, b).denom * c) = r.num * reduce(a, b).denom * c
    a * r.denom = r.denom * a
    r.denom * reduce(a, b).denom = reduce(a, b).denom * r.denom
    r + reduce(a, b) = reduce(r.num * reduce(a, b).denom * c + r.denom * reduce(a, b).num * c,
                              reduce(a, b).denom * r.denom * c)
}

theorem add_reduced(a: Int, b: Int, c: Int, d: Int) {
    b != Int.0 and d != Int.0 implies reduce(a, b) + reduce(c, d) = reduce(a * d + b * c, b * d)
} by {
    let e: Int satisfy {
        reduce(a, b).num * e = a and reduce(a, b).denom * e = b
    }
    reduce(reduce(a, b).num * d + c * reduce(a, b).denom, reduce(a, b).denom * d) = reduce(
        (reduce(a, b).num * d + c * reduce(a, b).denom) * e, reduce(a, b).denom * d * e)
    reduce(reduce(a, b).num * d + c * reduce(a, b).denom, reduce(a, b).denom * d) = reduce(a, b) + reduce(c, d) or Int.0 = d
    reduce(a, b).num * d * e + c * reduce(a, b).denom * e = (reduce(a, b).num * d + c * reduce(a, b).denom) * e
    c * (reduce(a, b).denom * e) = c * reduce(a, b).denom * e
    d * (reduce(a, b).num * e) = d * reduce(a, b).num * e
    e * (reduce(a, b).denom * d) = e * reduce(a, b).denom * d
    c * (reduce(a, b).denom * e) = reduce(a, b).denom * e * c
    d * (reduce(a, b).num * e) = reduce(a, b).num * e * d
    d * reduce(a, b).num = reduce(a, b).num * d
    e * (reduce(a, b).denom * d) = reduce(a, b).denom * d * e
    e * reduce(a, b).denom = reduce(a, b).denom * e
    reduce(a, b) + reduce(c, d) = reduce(reduce(a, b).num * e * d + reduce(a, b).denom * e * c,
                                         reduce(a, b).denom * e * d)
}

theorem add_reduced_same_denom(a: Int, b: Int, c: Int) {
    reduce(a, c) + reduce(b, c) = reduce(a + b, c)
} by {
    if c = Int.0 {
    } else {
        reduce(a * c + c * b, c * c) = reduce(a, c) + reduce(b, c) or Int.0 = c or Int.0 = c
        a * c + b * c = (a + b) * c
        b * c = c * b
        reduce(a, c) + reduce(b, c) = reduce((a + b) * c, c * c)
        reduce(a, c) + reduce(b, c) = reduce(a + b, c)
    }
}

theorem add_assoc(a: Rat, b: Rat, c: Rat) {
    a + b + c = a + (b + c)
} by {
    let d: Int = a.denom * b.denom * c.denom
    let an: Int = a.num * (b.denom * c.denom)
    b.denom * c.denom != Int.0

    let bn: Int = b.num * (a.denom * c.denom)
    a.denom * c.denom != Int.0

    let cn: Int = c.num * (a.denom * b.denom)
    a.denom * b.denom != Int.0

    reduce(an + bn, d) + reduce(cn, d) = reduce(an + bn + cn, d)
    reduce(an, d) + reduce(bn, d) = reduce(an + bn, d)
    reduce(a.num * (b.denom * c.denom), a.denom * (b.denom * c.denom)) = reduce(a.num, a.denom) or b.denom * c.denom = Int.0
    reduce(b.num * (a.denom * c.denom), b.denom * (a.denom * c.denom)) = reduce(b.num, b.denom) or a.denom * c.denom = Int.0
    reduce(c.num * (a.denom * b.denom), c.denom * (a.denom * b.denom)) = reduce(c.num, c.denom) or a.denom * b.denom = Int.0
    a.denom * (b.denom * c.denom) = a.denom * b.denom * c.denom
    b.denom * (a.denom * c.denom) = b.denom * a.denom * c.denom
    an + (bn + cn) = an + bn + cn
    reduce(a.num, a.denom) = a
    reduce(b.num, b.denom) = b
    reduce(c.num, c.denom) = c
    b.denom * a.denom = a.denom * b.denom
    c.denom * (a.denom * b.denom) = a.denom * b.denom * c.denom
    reduce(an, d) + reduce(bn + cn, d) = reduce(an + (bn + cn), d)
    reduce(bn, d) + reduce(cn, d) = reduce(bn + cn, d)
    a + b + c = reduce(an + (bn + cn), d)
}

theorem common_denom(r1: Rat, r2: Rat) {
    exists(n1: Int, n2: Int, d: Int) {
        r1 = reduce(n1, d) and r2 = reduce(n2, d)
    }
} by {
    let d: Int = r1.denom * r2.denom
    let n1: Int = r1.num * r2.denom
    r1 = reduce(r1.num * r2.denom, r1.denom * r2.denom)
    let n2: Int = r2.num * r1.denom
    r2 = reduce(r2.num * r1.denom, r2.denom * r1.denom)
}

theorem distrib_left(r1: Rat, r2: Rat, r3: Rat) {
    r1 * (r2 + r3) = r1 * r2 + r1 * r3
} by {
    let (a: Int, b: Int, c: Int) satisfy {
        r2 = reduce(a, c) and r3 = reduce(b, c)
    }

    // Simplify the left side
    r1 * (r2 + r3) = reduce(r1.num * a + r1.num * b, r1.denom * c)

    // Simplify the right side
    reduce(r1.num, r1.denom) * reduce(a, c) = reduce(r1.num * a, r1.denom * c)
    reduce(r1.num, r1.denom) * reduce(b, c) = reduce(r1.num * b, r1.denom * c)
    reduce(r1.num * b, r1.denom * c) + reduce(r1.num * a, r1.denom * c) = reduce(r1.num * b + r1.num * a, r1.denom * c)
    r1 * r2 = r2 * r1
    r1 * r3 + r2 * r1 = r2 * r1 + r1 * r3
    reduce(r1.num, r1.denom) = r1
    r1.num * b + r1.num * a = r1.num * a + r1.num * b
}

theorem distrib_right(r1: Rat, r2: Rat, r3: Rat) {
    (r1 + r2) * r3 = r1 * r3 + r2 * r3
}

theorem add_inv_cancels_right(a: Rat, b: Rat) {
    a + -a = Rat.0
} by {
    a.num * a.denom + -a.num * a.denom = (a.num + -a.num) * a.denom
    reduce(a.num * (-a).denom + (-a).num * a.denom, a.denom * (-a).denom) = a + -a
    -a.num = (-a).num
    a.num + -a.num = Int.0
    Int.0 * a.denom = Int.0
    reduce(Int.0, a.denom * (-a).denom) = Rat.0
    (-a).denom = a.denom
}

theorem add_inv_cancels_left(a: Rat, b: Rat) {
    -a + a = Rat.0
}

// The additive algebraic structure.

from algebra.add_semigroup import AddSemigroup

instance Rat: AddSemigroup

from algebra.add_comm_semigroup import AddCommSemigroup

instance Rat: AddCommSemigroup

from algebra.add_monoid import AddMonoid

instance Rat: AddMonoid

from algebra.add_comm_monoid import AddCommMonoid

instance Rat: AddCommMonoid

from algebra.add_group import AddGroup

instance Rat: AddGroup

from algebra.add_comm_group import AddCommGroup

instance Rat: AddCommGroup

theorem sub_self(a: Rat) {
    a - a = Rat.0
}

theorem reduce_one_one {
    reduce(Int.1, Int.1) = Rat.1
} by {
    Rat.from_int(Int.1).num = Int.1
    Rat.from_int(Int.1).denom = Int.1
    reduce(Rat.from_int(Int.1).num, Rat.from_int(Int.1).denom) = Rat.from_int(Int.1)
}

theorem reduce_self(n: Int) {
    n != Int.0 implies reduce(n, n) = Rat.1
}

theorem mul_inv_cancels_right(a: Rat) {
    a != Rat.0 implies a * a.inverse = Rat.1
} by {
    a * a.inverse = reduce(a.num * a.denom, a.num * a.denom)
    a.num * a.denom != Int.0 or a.num = Int.0 or a.denom = Int.0
    reduce(a.num * a.denom, a.num * a.denom) = Rat.1 or a.num * a.denom = Int.0
    reduce(a.num, a.denom) = a
    reduce(Int.0, a.denom) = Rat.0
    a.denom != Int.0
}

theorem mul_inv_cancels_left(a: Rat) {
    a != Rat.0 implies a.inverse * a = Rat.1
}

theorem sub_zero(a: Rat) {
    a - Rat.0 = a
}

theorem not_pos_and_neg(a: Rat) {
    a.is_positive implies not a.is_negative
}

theorem reduce_pos_pos(a: Int, b: Int) {
    a.is_positive and b.is_positive implies reduce(a, b).is_positive
} by {
    b != Int.0
    let d: Int satisfy {
        reduce(a, b).num * d = a and reduce(a, b).denom * d = b
    }
    reduce(a, b).denom.is_positive
    d.is_positive
}

theorem add_pos_pos(a: Rat, b: Rat) {
    a.is_positive and b.is_positive implies (a + b).is_positive
} by {
    (b.num * a.denom).is_positive
    (a.num * b.denom + b.num * a.denom).is_positive
    (a.denom * b.denom).is_positive
}

theorem mul_pos_pos(a: Rat, b: Rat) {
    a.is_positive and b.is_positive implies (a * b).is_positive
} by {
    (a.denom * b.denom).is_positive
    reduce(a.num * b.num, a.denom * b.denom) = a * b
    not (a.num * b.num).is_positive or not (a.denom * b.denom).is_positive or reduce(a.num * b.num, a.denom * b.denom).is_positive
    not a.num.is_positive or not b.num.is_positive or (a.num * b.num).is_positive
    a.num.is_positive = a.is_positive
    b.num.is_positive = b.is_positive
}

theorem pos_inverse(a: Rat) {
    a.is_positive implies a.inverse.is_positive
}

theorem add_cancels_sub(a: Rat, b: Rat) {
    a - b + b = a
}

theorem neg_neg_is_pos(a: Rat) {
    a.is_negative implies (-a).is_positive
}

theorem neg_pos_is_neg(a: Rat) {
    a.is_positive implies (-a).is_negative
}

theorem zero_minus(a: Rat) {
    Rat.0 - a = -a
}

theorem mul_cancels_div(a: Rat, b: Rat) {
    b != Rat.0 implies (a / b) * b = a
}

/// True if this rational is less than or equal to the other.
instance Rat: LTE {
    define lte(self, other: Rat) -> Bool {
        (other - self).is_positive or self = other
    }
}

from data.basic.relation_basic import is_reflexive, is_transitive, is_antisymmetric, transitive_step

theorem rat_is_reflexive {
    is_reflexive(Rat.lte)
}

theorem lte_trans(a: Rat, b: Rat, c: Rat) {
    a <= b and b <= c implies a <= c
} by {
    if a = b {
    } else {
        if b = c {
            a <= c
        } else {
            ((c - b) + (b - a)).is_positive
            a <= c
        }
    }
}

theorem rat_is_transitive {
    is_transitive(Rat.lte)
} by {
    forall(a: Rat, b: Rat, c: Rat) {
    }
}

theorem lte_antisymm(a: Rat, b: Rat) {
    a <= b and b <= a implies a = b
} by {
    if a = b {
    } else {
        ((b - a) + (a - b)).is_positive
    }
}

theorem rat_is_antisymmetric {
    is_antisymmetric(Rat.lte)
}

from order import PartialOrder
from order_relation import lt_relation_is_transitive

instance Rat: PartialOrder

theorem pos_imp_zero_lt(a: Rat) {
    a.is_positive implies Rat.0 < a
}

theorem zero_lt_imp_pos(a: Rat) {
    Rat.0 < a implies a.is_positive
}

theorem lt_trans(a: Rat, b: Rat, c: Rat) {
    a < b and b < c implies a < c
} by {
    if a < b and b < c {
        lt_relation_is_transitive[Rat]
        transitive_step(Rat.lt, a, b, c)
        a < c
    }
}

theorem neg_imp_lt_zero(a: Rat) {
    a.is_negative implies a < Rat.0
} by {
    (-a).is_positive
    Rat.0 - a = -a
    a <= Rat.0
    a != Rat.0
}

theorem lt_zero_imp_neg(a: Rat) {
    a < Rat.0 implies a.is_negative
}

theorem two_neq_zero {
    Rat.2 != Rat.0
}

theorem times_two(r: Rat) {
    Rat.2 * r = r + r
}

theorem mul_cancels_right(a: Rat, b: Rat, c: Rat) {
    c != Rat.0 and a * c = b * c implies a = b
} by {
    c.inverse * (c * a) = c.inverse * c * a
    c.inverse * (c * b) = c.inverse * c * b
    c * c.inverse = Rat.1 or Rat.0 = c
    a * c = c * a
    b * c = c * b
    c * c.inverse = c.inverse * c
}

theorem half_plus_half(r: Rat) {
    (r / Rat.2) + (r / Rat.2) = r
}

theorem neg_mul(a: Rat, b: Rat) {
    (-a) * b = -(a * b)
}

theorem mul_neg_pos(a: Rat, b: Rat) {
    a.is_negative and b.is_positive implies (a * b).is_negative
} by {
    -a * b = -(a * b)
    not b.is_positive or not (-a).is_positive or (b * -a).is_positive
    b * -a = -a * b
    (-(a * b)).num = -(a * b).num
    not a.is_negative or (-a).is_positive
    (a * b).num.is_negative = (a * b).is_negative
    (b * -a).num.is_positive = (b * -a).is_positive
    (--(a * b).num).is_negative = (-(a * b).num).is_positive
    --(a * b).num = (a * b).num
}

theorem mul_pos_neg(a: Rat, b: Rat) {
    a.is_positive and b.is_negative implies (a * b).is_negative
}

theorem mul_neg_neg(a: Rat, b: Rat) {
    a.is_negative and b.is_negative implies (a * b).is_positive
}

theorem one_is_pos {
    Rat.1.is_positive
}

theorem two_is_pos {
    Rat.2.is_positive
}

theorem half_is_pos {
    Rat.2.inverse.is_positive
}

theorem sub_add_quasi_cancel(a: Rat, b: Rat) {
    a - (a + b) = -b
}

theorem not_lt_self(a: Rat) {
    not a < a
}

theorem not_lt_both_ways(a: Rat, b: Rat) {
    a < b implies not b < a
}

theorem lt_add_pos(a: Rat, b: Rat) {
    b.is_positive implies a < a + b
} by {
    (a + b) - a = b
    Rat.0 < b
    a <= a + b
    a != a + b
}

attributes Rat {
    /// The absolute value of a rational number.
    define abs(self) -> Rat {
        if self.is_negative {
            -self
        } else {
            self
        }
    }
}

theorem neg_sub(a: Rat, b: Rat) {
    -(a - b) = b - a
}

theorem single_trichotomy(a: Rat) {
    a.is_positive or a.is_negative or a = Rat.0
} by {
    a.num.is_positive or a.num.is_negative or a.num = Int.0
    a.num.is_positive = a.is_positive
    a.num.is_negative = a.is_negative
    if a.num.is_positive {
        a.is_positive
    } else {
        if a.num.is_negative {
            a.is_negative
        } else {
            a.num = Int.0
            reduce(Int.0, a.denom) = Rat.0
            reduce(a.num, a.denom) = a
            a = Rat.0
        }
    }
}

theorem trichotomy(a: Rat, b: Rat) {
    a < b or a = b or a > b
} by {
    a > b = b < a
    -(a - b) = b - a
    if (a - b).is_positive {
        b <= a
        b < a
        a > b
    } else {
        if (a - b).is_negative {
            (-(a - b)).is_positive
            (b - a).is_positive
            a <= b
            a < b
        } else {
            a = b
        }
    }
}

theorem lt_add_right(a: Rat, b: Rat, c: Rat) {
    a < b implies a + c < b + c
} by {
    (b + c) - (a + c) = b + (c + -c) + -a
    ((b + c) - (a + c)).is_positive
}

theorem add_neg_lt(a: Rat, b: Rat) {
    a.is_negative implies b + a < b
}

theorem adding_lts(a: Rat, b: Rat, c: Rat, d: Rat) {
    a < b and c < d implies a + c < b + d
} by {
    a + c < b + c
    c + b = b + c
    c + b < d + b
    b + c < b + d
    a + c < b + d
}

theorem minus_cancels_plus(a: Rat, b: Rat) {
    a + b - b = a
}

theorem gt_minus_pos(a: Rat, b: Rat) {
    b.is_positive implies a > a - b
}

theorem sub_add(a: Rat, b: Rat, c: Rat) {
    a - (b + c) = a - b - c
}

theorem no_greatest(a: Rat) {
    exists(b: Rat) {
        a < b
    }
}

theorem sub_from_int(a: Int, b: Int) {
    Rat.from_int(a) - Rat.from_int(b) = Rat.from_int(a - b)
}

theorem lt_from_int(a: Int, b: Int) {
    a < b implies Rat.from_int(a) < Rat.from_int(b)
} by {
    Rat.from_int(a) != Rat.from_int(b)
    Rat.from_int(b) - Rat.from_int(a) = Rat.from_int(b - a)
    not a <= b or (b - a).is_positive or b = a
    not Rat.from_int(a) <= Rat.from_int(b) or Rat.from_int(a) < Rat.from_int(b) or Rat.from_int(b) = Rat.from_int(a)
    not (Rat.from_int(b) - Rat.from_int(a)).is_positive or Rat.from_int(a) <= Rat.from_int(b)
    not a < b or a <= b
    (Rat.from_int(b) - Rat.from_int(a)).num.is_positive = (Rat.from_int(b) - Rat.from_int(a)).is_positive
    Rat.from_int(b - a).num = b - a
}

theorem lte_from_int(a: Int, b: Int) {
    a <= b implies Rat.from_int(a) <= Rat.from_int(b)
} by {
    if a = b {
        Rat.from_int(a) = Rat.from_int(b)
        Rat.from_int(a) <= Rat.from_int(b)
    } else {
        a < b
        Rat.from_int(a) < Rat.from_int(b)
        Rat.from_int(a) < Rat.from_int(b) = (Rat.from_int(a) <= Rat.from_int(b) and Rat.from_int(a) != Rat.from_int(b))
        Rat.from_int(a) <= Rat.from_int(b)
    }
}

theorem gt_from_int(a: Int, b: Int) {
    a > b implies Rat.from_int(a) > Rat.from_int(b)
}

theorem gte_from_int(a: Int, b: Int) {
    a >= b implies Rat.from_int(a) >= Rat.from_int(b)
}

theorem lt_mul_pos(a: Rat, b: Rat, c: Rat) {
    a < b and c.is_positive implies a * c < b * c
} by {
    b * c + -a * c = (b + -a) * c
    not a <= b or (b - a).is_positive or b = a
    -a * c = -(a * c)
    not (b - a).is_positive or not c.is_positive or ((b - a) * c).is_positive
    b * c + -(a * c) = b * c - a * c
    b + -a = b - a
    (b + -a) * c > Rat.0 = Rat.0 < (b + -a) * c
    not a < b or a <= b
    not a < b or b != a
    not ((b - a) * c).is_positive or Rat.0 < (b - a) * c
    b * c - a * c > Rat.0
    a * c <= b * c
}

theorem lte_mul_pos(a: Rat, b: Rat, c: Rat) {
    a <= b and c.is_positive implies a * c <= b * c
} by {
    a * c < b * c or a * c = b * c
}

theorem lt_div_pos(a: Rat, b: Rat, c: Rat) {
    c.is_positive and a < b implies a / c < b / c
}

theorem lte_div_pos(a: Rat, b: Rat, c: Rat) {
    c.is_positive and a <= b implies a / c <= b / c
}

theorem mul_div_cancels(a: Rat, b: Rat) {
    b != Rat.0 implies (a * b) / b = a
}

theorem div_from_int(a: Rat) {
    Rat.from_int(a.num) / Rat.from_int(a.denom) = a
} by {
    Rat.from_int(a.num) / Rat.from_int(a.denom) = reduce(a.num, Int.1) * reduce(Int.1, a.denom)
}

theorem lt_from_int_mul_denom(a: Rat, n: Int) {
    a.num < n * a.denom implies a < Rat.from_int(n)
} by {
    Rat.from_int(a.denom).is_positive
    Rat.from_int(n) * Rat.from_int(a.denom) / Rat.from_int(a.denom) = Rat.from_int(n)
}

theorem lte_from_int_mul_denom(a: Rat, n: Int) {
    a.num <= n * a.denom implies a <= Rat.from_int(n)
} by {
    if a.num = n * a.denom {
        a = Rat.from_int(n)
    } else {
        a.num < n * a.denom
        a < Rat.from_int(n)
    }
}

theorem floor_exists(a: Rat) {
    exists(q: Int) {
        Rat.from_int(q) <= a and a < Rat.from_int(q + Int.1)
    }
} by {
    a.denom.is_positive

    // This should be a resolution from the "division_theorem" theorem.
    let (q: Int, r: Int) satisfy {
        Int.0 <= r and r < a.denom and a.num = q * a.denom + r
    }

    // Left bound
    not Int.0 <= r or q * a.denom + Int.0 <= q * a.denom + r
    Rat.from_int(q * a.denom) + Rat.from_int(Int.0) = Rat.from_int(q * a.denom + Int.0)
    Rat.from_int(q * a.denom) + Rat.from_int(r) = Rat.from_int(q * a.denom + r)
    not q * a.denom + r >= q * a.denom + Int.0 or Rat.from_int(q * a.denom + r) >= Rat.from_int(q * a.denom + Int.0)
    not q * a.denom + Int.0 <= q * a.denom + r or q * a.denom + r >= q * a.denom + Int.0
    Rat.from_int(q * a.denom) + Rat.from_int(r) >= Rat.from_int(q * a.denom) + Rat.from_int(Int.0)
    Rat.from_int(q) * Rat.from_int(a.denom) / Rat.from_int(a.denom) <= Rat.from_int(a.num) / Rat.from_int(a.denom)
    Rat.from_int(q) * Rat.from_int(a.denom) / Rat.from_int(a.denom) = Rat.from_int(q)

    // Right bound
    a.num < (q + Int.1) * a.denom
}

let floor_impl(a: Rat) -> q: Int satisfy {
    Rat.from_int(q) <= a and a < Rat.from_int(q + Int.1)
}

attributes Rat {
    /// The floor of a rational number (the greatest integer less than or equal to it).
    define floor(self) -> Int {
        floor_impl(self)
    }
}

theorem mul_lt_lt(a: Rat, b: Rat, c: Rat, d: Rat) {
    a.is_positive and c.is_positive and a < b and c < d implies a * c < b * d
} by {
    b.is_positive
}

theorem mul_lt_lte(a: Rat, b: Rat, c: Rat, d: Rat) {
    a.is_positive and c.is_positive and a < b and c <= d implies a * c < b * d
}

theorem pos_lte(a: Rat, b: Rat) {
    a.is_positive and a <= b implies b.is_positive
} by {
    not a <= b or (b - a).is_positive or b = a
    not (b - a).is_positive or not a.is_positive or (b - a + a).is_positive
    b - a + a = b
}

theorem mul_lte_lt(a: Rat, b: Rat, c: Rat, d: Rat) {
    a.is_positive and c.is_positive and a <= b and c < d implies a * c < b * d
}

theorem lt_pos_inverse(a: Rat, b: Rat) {
    a.is_positive and a < b implies b.inverse < a.inverse
} by {
    if a.inverse <= b.inverse {
        a.is_positive
        a < b
        a <= b
        pos_lte(a, b)
        b.is_positive
        pos_inverse(a)
        a.inverse.is_positive
        pos_inverse(b)
        b.inverse.is_positive
        lte_mul_pos(a.inverse, b.inverse, a)
        a.inverse * a <= b.inverse * a
        lt_mul_pos(a, b, b.inverse)
        a * b.inverse < b * b.inverse
        mul_comm(a, b.inverse)
        a * b.inverse = b.inverse * a
        mul_comm(b, b.inverse)
        b * b.inverse = b.inverse * b
        a.inverse * a = Rat.1
        b.inverse * b = Rat.1
        Rat.1 <= b.inverse * a
        b.inverse * a < Rat.1
        false
    }
}

theorem inverse_inverts(a: Rat) {
    a != Rat.0 implies a.inverse.inverse = a
} by {
    a * a.inverse = Rat.1 or Rat.0 = a
    a * a.inverse / a.inverse = a or a.inverse = Rat.0
    a * a.inverse * a.inverse.inverse = a * a.inverse / a.inverse
    a.inverse * a = Rat.1
    if a.inverse = Rat.0 {
        Rat.0 * a = Rat.0
        a.inverse * a = Rat.0 * a
        a.inverse * a = Rat.0
        false
    }
    a.inverse * a / a.inverse = a
    Rat.1 / a.inverse = a
    Rat.1 / a.inverse = Rat.1 * a.inverse.inverse
    Rat.1 * a.inverse.inverse = a
}

theorem smaller_int_inverse(a: Rat) {
    a.is_positive implies exists(n: Int) {
        n.is_positive and Rat.from_int(n).inverse < a
    }
} by {
    let n: Int satisfy {
        a.inverse < Rat.from_int(n)
    }
    a.inverse.inverse = a
    n.is_positive
}

theorem gte_some_int(a: Rat) {
    exists(n: Int) {
        a >= Rat.from_int(n)
    }
}

theorem mul_denom(r: Rat) {
    r * Rat.from_int(r.denom) = Rat.from_int(r.num)
}

theorem half_pos(r: Rat) {
    r.is_positive implies (r / Rat.2).is_positive
}

theorem add_half_half(r: Rat) {
    (r / Rat.2) + (r / Rat.2) = r
}

theorem lt_neg(p: Rat, q: Rat) {
    p < q implies -q < -p
} by {
    (-p - -q).is_positive
}

theorem lt_lte_trans(a: Rat, b: Rat, c: Rat) {
    a < b and b <= c implies a < c
} by {
    if b = c {
    } else {
    }
}

theorem lte_lt_trans(a: Rat, b: Rat, c: Rat) {
    a <= b and b < c implies a < c
} by {
    if a = b {
    } else {
    }
}

theorem lt_imp_rat_between(a: Rat, b: Rat) {
    a < b implies exists(c: Rat) {
        a < c and c < b
    }
} by {
    a / Rat.2 < b / Rat.2
    not a / Rat.2 < b / Rat.2 or a / Rat.2 + a / Rat.2 < b / Rat.2 + a / Rat.2
    a / Rat.2 + a / Rat.2 = a
    b / Rat.2 + a / Rat.2 = a / Rat.2 + b / Rat.2
    a / Rat.2 + b / Rat.2 < b
}

theorem gt_imp_rat_between(a: Rat, b: Rat) {
    a > b implies exists(c: Rat) {
        a > c and c > b
    }
} by {
    if a > b {
        a > b = b < a
        let c: Rat satisfy {
            b < c and c < a
        }
        a > c = c < a
        c > b = b < c
        a > c and c > b
    }
}

theorem lt_cancel_pos_mul_right(a: Rat, b: Rat, c: Rat) {
    c.is_positive and a * c < b * c implies a < b
}

theorem lt_cancel_pos_mul_left(a: Rat, b: Rat, c: Rat) {
    c.is_positive and c * a < c * b implies a < b
}

theorem gt_cancel_pos_mul_right(a: Rat, b: Rat, c: Rat) {
    c.is_positive and a * c > b * c implies a > b
}

theorem gt_cancel_pos_mul_left(a: Rat, b: Rat, c: Rat) {
    c.is_positive and c * a > c * b implies a > b
}

theorem cancel_positivity_left(a: Rat, b: Rat) {
    a.is_positive and (a * b).is_positive implies b.is_positive
} by {
    if b.is_negative {
        false
    }
}

theorem cancel_positivity_right(a: Rat, b: Rat) {
    b.is_positive and (a * b).is_positive implies a.is_positive
}

theorem mul_cancels_div_left(a: Rat, b: Rat) {
    b != Rat.0 implies b * (a / b) = a
}

theorem neg_abs(a: Rat) {
    (-a).abs = a.abs
} by {
    if a.is_negative {
    } else {
        if a = Rat.0 {
            (-a).abs = a.abs
        } else {
            (-a).abs = a.abs
        }
    }
}

theorem abs_non_pos(a: Rat) {
    not a.is_positive implies a.abs = -a
}

theorem abs_non_neg(a: Rat) {
    not a.is_negative implies a.abs = a
}

theorem abs_mul_abs(a: Rat, b: Rat) {
    (a * b).abs = (a * b.abs).abs
} by {
    if b.is_positive {
    } else {
        b.abs = b or b.is_negative
        not b.is_negative or -b = b.abs
        a * b.abs = b.abs * a
        --b = b
        a * --b = --b * a
        -(-b * a) = --b * a
        (-(-b * a)).abs = (-b * a).abs
    }
}

theorem mul_non_neg(a: Rat, b: Rat) {
    not a.is_negative and not b.is_negative implies not (a * b).is_negative
} by {
    if a = Rat.0 {
    } else {
        if b = Rat.0 {
            not (a * b).is_negative
        } else {
            not (a * b).is_negative
        }
    }
}

theorem mul_two_abs(a: Rat, b: Rat) {
    a.abs * b.abs = (a * b).abs
} by {
    not b.abs.is_negative
    not (a.abs * b.abs).is_negative
}

theorem abs_zero_imp_zero(a: Rat) {
    a.abs = Rat.0 implies a = Rat.0
} by {
    a.abs = a or a.is_negative
    not a.is_negative or -a = a.abs
    not a.is_negative or (-a).is_positive
    not a.abs.is_positive or Rat.0 < a.abs
    not Rat.0 < a.abs or Rat.0 != a.abs
}

theorem zero_lte_abs(a: Rat) {
    Rat.0 <= a.abs
} by {
    if a.abs.is_negative {
        not a.abs.is_negative or -a.abs = a.abs.abs
        not a.abs.is_negative or (-a.abs).is_positive
        (-a.abs).is_positive
        a.abs.abs = -a.abs or a.abs.is_positive
        a.abs.abs = -a.abs
        a.abs.abs.is_positive
        a.abs.abs = a.abs or a.abs.is_negative
        a.abs.abs = a.abs
        a.abs.is_positive
        false
    } else {
        if a.abs = Rat.0 {
            forall(x: Rat) {
                x <= x
            }
            a.abs <= a.abs
            a.abs = Rat.0 or Rat.0 <= a.abs
            Rat.0 <= a.abs
        } else {
            a.abs.is_negative or a.abs.is_positive or a.abs = Rat.0
            a.abs.is_positive
            Rat.0 < a.abs
            Rat.0 <= a.abs
        }
    }
}

theorem pos_inverses_lt(p: Rat, q: Rat, r: Rat) {
    p.is_positive and q.is_positive and r.is_positive
    and r / p < r / q
    implies
    q < p
} by {
    r / p * p = r
    not p.is_positive or not r / p < r / q or r / p * p < r / q * p
    not q.is_positive or not r < p * r / q or r * q < p * r / q * q
    p * (r * q.inverse) = p * r * q.inverse
    p * r * q.inverse = p * r / q
    r * q.inverse = r / q
    p * (r / q) = r / q * p
    r * q < p * r / q * q
    (p * r) / q * q = p * r
}

theorem inverses_eq(p: Rat, q: Rat, r: Rat) {
    p != Rat.0 and q != Rat.0 and r != Rat.0
    and r / p = r / q
    implies
    q = p
}

theorem pos_inverses_lte(p: Rat, q: Rat, r: Rat) {
    p.is_positive and q.is_positive and r.is_positive
    and r / p <= r / q
    implies
    q <= p
} by {
    if r / p = r / q {
        p != Rat.0
        q != Rat.0
        q <= p
    } else {
        q < p
    }
}

theorem pos_inverses_gt(p: Rat, q: Rat, r: Rat) {
    p.is_positive and q.is_positive and r.is_positive
    and r / p > r / q
    implies
    q > p
}

theorem pos_inverses_gte(p: Rat, q: Rat, r: Rat) {
    p.is_positive and q.is_positive and r.is_positive
    and r / p >= r / q
    implies
    q >= p
}

theorem lt_cancel_add_right(p: Rat, q: Rat, r: Rat) {
    p + r < q + r implies p < q
}

theorem sub_distrib(p: Rat, q: Rat, r: Rat) {
    p * (q - r) = p * q - p * r
} by {
    -r * p + q * p = (-r + q) * p
    --r * p = -(-r * p)
    p * q + -(p * r) = p * q - p * r
    q + -r = q - r
    p * q + -(p * r) = -(p * r) + p * q
    q + -r = -r + q
    (q - r) * p = p * (q - r)
    q * p = p * q
    r * p = p * r
    --(-r * p) = -r * p
    --r = r
}

theorem half_lt_one {
    Rat.2.inverse < Rat.1
} by {
    Rat.1 / Rat.2 + Rat.1 / Rat.2 = Rat.1
    not (Rat.1 / Rat.2).is_positive or Rat.2.inverse < Rat.2.inverse + Rat.1 / Rat.2
    Rat.1 * Rat.2.inverse = Rat.1 / Rat.2
    Rat.1 * Rat.2.inverse = Rat.2.inverse
}

theorem lower_squared(a: Rat) {
    a.is_positive
    implies
    exists(eps: Rat) {
        eps.is_positive and eps * eps < a
    }
} by {
    if Rat.1 <= a {
        let eps: Rat = Rat.2.inverse
        eps.is_positive and eps * eps < a
    } else {
        a < Rat.1
        let eps: Rat = a * a
        eps.is_positive
        eps.is_positive and eps * eps < a
    }
}

theorem square_lt_imp_lt(a: Rat, b: Rat) {
    a.is_positive and b.is_positive and a * a < b * b
    implies
    a < b
} by {
    if a >= b {
        false
    }
}

theorem smaller_positive(a: Rat) {
    a.is_positive implies
    exists(r: Rat) {
        r.is_positive and r < a
    }
}

theorem lt_both_pos(a: Rat, b: Rat) {
    a.is_positive and b.is_positive implies
    exists(r: Rat) {
        r.is_positive and r < a and r < b
    }
} by {
    if a < b {
        let r: Rat satisfy {
            r.is_positive and r < a
        }
        r.is_positive and r < a and r < b
    } else {
        let r: Rat satisfy {
            r.is_positive and r < b
        }
        r.is_positive and r < a and r < b
    }
}

theorem lt_rhs_div_pos(a: Rat, b: Rat, c: Rat) {
    c.is_positive and a < b / c implies a * c < b
} by {
    b / c * c = b
}

attributes Rat {
    /// True if the absolute difference between two rationals is less than epsilon.
    define is_close(self, other: Rat, eps: Rat) -> Bool {
        (self - other).abs < eps
    }
}

theorem lte_abs(q: Rat) {
    q <= q.abs
}

theorem close_comm(a: Rat, b: Rat, eps: Rat) {
    a.is_close(b, eps) implies b.is_close(a, eps)
}

theorem close_imp_bounds(a: Rat, b: Rat, eps: Rat) {
    a.is_close(b, eps) implies a < b + eps and a > b - eps
} by {
    // Left ineq
    (a - b).abs < eps = a.is_close(b, eps)
    not a - b < eps or a - b + b < eps + b
    not a - b <= (a - b).abs or not (a - b).abs < eps or a - b < eps
    a - b <= (a - b).abs
    a - b + b < eps + b

    // Right ineq
    b - a < eps
    eps + a > b
    not b - eps + eps < a + eps or b - eps < a
    a > b - eps = b - eps < a
    eps + a > b = b < eps + a
    a - b + b = a
    b - eps + eps = b
    eps + a = a + eps
    eps + b = b + eps
}

theorem bounds_imp_close(a: Rat, b: Rat, eps: Rat) {
    a < b + eps and a > b - eps implies a.is_close(b, eps)
} by {
    if (a - b).is_negative {
        not b - eps < a or b - eps + eps < a + eps
        eps + a = a + eps
        a > b - eps = b - eps < a
        b - a + a = b
        b - eps + eps = b
        b - a + a < eps + a
    } else {
        a - b + b < eps + b
        a.is_close(b, eps)
    }
}

attributes Rat {
    /// Converts a natural number to a rational number.
    let from_nat: Nat -> Rat = function(n: Nat) {
        Rat.from_int(Int.from_nat(n))
    }
}

theorem nat_lt_imp_rat_lt(a: Nat, b: Nat) {
    a < b implies Rat.from_nat(a) < Rat.from_nat(b)
}

theorem from_nat_nonneg(n: Nat) {
    Rat.0 <= Rat.from_nat(n)
} by {
    Nat.0 <= n
    Int.from_nat(Nat.0) <= Int.from_nat(n)
    Rat.from_int(Int.from_nat(Nat.0)) <= Rat.from_int(Int.from_nat(n))
    Rat.from_nat(Nat.0) = Rat.0
    Rat.from_int(Int.from_nat(n)) = Rat.from_nat(n)
}

/// The embedding of natural numbers into rationals preserves addition.
theorem from_nat_add(a: Nat, b: Nat) {
    Rat.from_nat(a) + Rat.from_nat(b) = Rat.from_nat(a + b)
} by {
    Int.from_nat(a) + Int.from_nat(b) = Int.from_nat(a + b)
}

/// The embedding of natural numbers into rationals preserves multiplication.
theorem from_nat_mul(a: Nat, b: Nat) {
    Rat.from_nat(a) * Rat.from_nat(b) = Rat.from_nat(a * b)
} by {
    Rat.from_int(Int.from_nat(a)) * Rat.from_int(Int.from_nat(b)) = Rat.from_int(Int.from_nat(a) * Int.from_nat(b))
    Int.from_nat(a) * Int.from_nat(b) = Int.from_nat(a * b)
}

/// The embedding of natural numbers into rationals preserves non-strict inequality.
theorem nat_lte_imp_rat_lte(a: Nat, b: Nat) {
    a <= b implies Rat.from_nat(a) <= Rat.from_nat(b)
}

theorem recip_eq_one_div(a: Rat) {
    a != Rat.0 implies a.inverse = Rat.1 / a
}
 
