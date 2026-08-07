from nat import Nat
from nat import add_cancels_right, add_sub
from int import Int, pos_is_not_neg, zero_not_pos
from pair import Pair, pair_eta, pair_new_first, pair_new_second
from data.basic.functions import inverse_fn, inverse_fn_apply_of_surjective, is_injective_fn,
    is_surjective_fn, surjective_fn_has_preimage
from rat.rat_base import Rat, reduce, reduce_idempotent, denom_positive,
    from_int_cancel

numerals Nat
numerals Int

/// The rational represented by an integer numerator and a positive natural
/// denominator encoded as its predecessor.
define rat_pair_enum(pair: Pair[Int, Nat]) -> Rat {
    reduce(pair.first, Int.from_nat(pair.second.suc))
}

/// The standard zig-zag enumeration of the integers:
/// `0, -1, 1, -2, 2, ...`.
define int_zigzag(n: Nat) -> Int {
    match n {
        Nat.zero {
            Int.0
        }
        Nat.suc(pred) {
            match int_zigzag(pred) {
                Int.from_nat(k) {
                    Int.neg_suc(k)
                }
                Int.neg_suc(k) {
                    Int.from_nat(k.suc)
                }
            }
        }
    }
}

/// Stepping the zig-zag enumeration after a nonnegative value gives the next
/// negative value.
theorem int_zigzag_suc_of_from_nat(n: Nat, k: Nat) {
    int_zigzag(n) = Int.from_nat(k) implies int_zigzag(n.suc) = Int.neg_suc(k)
}

/// Stepping the zig-zag enumeration after a negative value gives the matching
/// positive successor.
theorem int_zigzag_suc_of_neg_suc(n: Nat, k: Nat) {
    int_zigzag(n) = Int.neg_suc(k) implies int_zigzag(n.suc) = Int.from_nat(k.suc)
}

/// Inductive predicate that the zig-zag enumeration hits both `k` and `-(k+1)`.
define int_zigzag_hits_pair(k: Nat) -> Bool {
    exists(nonneg_idx: Nat) {
        int_zigzag(nonneg_idx) = Int.from_nat(k)
    } and exists(neg_idx: Nat) {
        int_zigzag(neg_idx) = Int.neg_suc(k)
    }
}

/// The zig-zag enumeration hits zero and negative one.
theorem int_zigzag_hits_pair_zero {
    int_zigzag_hits_pair(Nat.0)
} by {
    int_zigzag(Nat.0) = Int.0
    Int.0 = Int.from_nat(Nat.0)
    exists(nonneg_idx: Nat) {
        int_zigzag(nonneg_idx) = Int.from_nat(Nat.0)
    }
    int_zigzag(Nat.0.suc) = Int.neg_suc(Nat.0)
    exists(neg_idx: Nat) {
        int_zigzag(neg_idx) = Int.neg_suc(Nat.0)
    }
}

/// If the zig-zag enumeration hits `k` and `-(k+1)`, it hits `k+1` and
/// `-(k+2)`.
theorem int_zigzag_hits_pair_step(k: Nat) {
    int_zigzag_hits_pair(k) implies int_zigzag_hits_pair(k.suc)
} by {
    if int_zigzag_hits_pair(k) {
        let neg_idx: Nat satisfy {
            int_zigzag(neg_idx) = Int.neg_suc(k)
        }
        int_zigzag_suc_of_neg_suc(neg_idx, k)
        int_zigzag(neg_idx.suc) = Int.from_nat(k.suc)
        exists(nonneg_idx: Nat) {
            int_zigzag(nonneg_idx) = Int.from_nat(k.suc)
        }
        int_zigzag_suc_of_from_nat(neg_idx.suc, k.suc)
        int_zigzag(neg_idx.suc.suc) = Int.neg_suc(k.suc)
        exists(next_neg_idx: Nat) {
            int_zigzag(next_neg_idx) = Int.neg_suc(k.suc)
        }
        int_zigzag_hits_pair(k.suc)
    }
}

/// The zig-zag enumeration hits every nonnegative and negative integer at the
/// corresponding natural magnitude.
theorem int_zigzag_hits_pair_all(k: Nat) {
    int_zigzag_hits_pair(k)
} by {
    int_zigzag_hits_pair_zero
    forall(j: Nat) {
        if int_zigzag_hits_pair(j) {
            int_zigzag_hits_pair_step(j)
            int_zigzag_hits_pair(j.suc)
        }
    }
}

/// Every integer appears in the zig-zag enumeration.
theorem int_zigzag_surjective {
    is_surjective_fn(int_zigzag)
} by {
    forall(z: Int) {
        match z {
            Int.from_nat(k) {
                int_zigzag_hits_pair_all(k)
                let n: Nat satisfy {
                    int_zigzag(n) = Int.from_nat(k)
                }
                exists(preimage: Nat) {
                    int_zigzag(preimage) = z
                }
            }
            Int.neg_suc(k) {
                int_zigzag_hits_pair_all(k)
                let n: Nat satisfy {
                    int_zigzag(n) = Int.neg_suc(k)
                }
                exists(preimage: Nat) {
                    int_zigzag(preimage) = z
                }
            }
        }
    }
}

/// Every integer has an explicit preimage under the zig-zag enumeration.
theorem int_zigzag_has_preimage(z: Int) {
    exists(n: Nat) {
        int_zigzag(n) = z
    }
} by {
    int_zigzag_surjective
    surjective_fn_has_preimage(int_zigzag, z)
}

/// The next pair in the diagonal enumeration of pairs of natural numbers.
define nat_pair_next(pair: Pair[Nat, Nat]) -> Pair[Nat, Nat] {
    match pair.second {
        Nat.zero {
            Pair.new(Nat.0, pair.first.suc)
        }
        Nat.suc(pred) {
            Pair.new(pair.first.suc, pred)
        }
    }
}

/// The diagonal enumeration of pairs of natural numbers:
/// `(0,0), (0,1), (1,0), (0,2), ...`.
define nat_pair_zigzag(n: Nat) -> Pair[Nat, Nat] {
    match n {
        Nat.zero {
            Pair.new(Nat.0, Nat.0)
        }
        Nat.suc(pred) {
            nat_pair_next(nat_pair_zigzag(pred))
        }
    }
}

/// The next pair after the end of a diagonal starts the next diagonal.
theorem nat_pair_next_zero_second(a: Nat) {
    nat_pair_next(Pair.new(a, Nat.0)) = Pair.new(Nat.0, a.suc)
} by {
    pair_new_first(a, Nat.0)
    pair_new_second(a, Nat.0)
}

/// The next pair inside a diagonal increments the first coordinate and
/// decrements the second coordinate.
theorem nat_pair_next_suc_second(a: Nat, b: Nat) {
    nat_pair_next(Pair.new(a, b.suc)) = Pair.new(a.suc, b)
} by {
    pair_new_first(a, b.suc)
    pair_new_second(a, b.suc)
}

/// Predicate that a pair of natural numbers occurs in the diagonal enumeration.
define nat_pair_zigzag_hits_pair(a: Nat, b: Nat) -> Bool {
    exists(n: Nat) {
        nat_pair_zigzag(n) = Pair.new(a, b)
    }
}

/// Occurrence in the pair enumeration is stable under equality of coordinates.
theorem nat_pair_zigzag_hits_pair_transport(a: Nat, b: Nat, c: Nat, d: Nat) {
    a = c and b = d and nat_pair_zigzag_hits_pair(a, b) implies
        nat_pair_zigzag_hits_pair(c, d)
} by {
    if a = c and b = d and nat_pair_zigzag_hits_pair(a, b) {
        let n: Nat satisfy {
            nat_pair_zigzag(n) = Pair.new(a, b)
        }
        Pair.new(a, b) = Pair.new(c, d)
        nat_pair_zigzag(n) = Pair.new(c, d)
        exists(preimage: Nat) {
            nat_pair_zigzag(preimage) = Pair.new(c, d)
        }
    }
}

/// The first pair occurs in the diagonal enumeration.
theorem nat_pair_zigzag_hits_zero_zero {
    nat_pair_zigzag_hits_pair(Nat.0, Nat.0)
} by {
    nat_pair_zigzag(Nat.0) = Pair.new(Nat.0, Nat.0)
    exists(n: Nat) {
        nat_pair_zigzag(n) = Pair.new(Nat.0, Nat.0)
    }
}

/// Occurrence propagates to the next point inside a diagonal.
theorem nat_pair_zigzag_hit_step_down(a: Nat, b: Nat) {
    nat_pair_zigzag_hits_pair(a, b.suc) implies
        nat_pair_zigzag_hits_pair(a.suc, b)
} by {
    if nat_pair_zigzag_hits_pair(a, b.suc) {
        let n: Nat satisfy {
            nat_pair_zigzag(n) = Pair.new(a, b.suc)
        }
        nat_pair_next_suc_second(a, b)
        nat_pair_zigzag(n.suc) = nat_pair_next(nat_pair_zigzag(n))
        nat_pair_zigzag(n.suc) = nat_pair_next(Pair.new(a, b.suc))
        nat_pair_zigzag(n.suc) = Pair.new(a.suc, b)
        exists(next_idx: Nat) {
            nat_pair_zigzag(next_idx) = Pair.new(a.suc, b)
        }
    }
}

/// Occurrence at the end of one diagonal gives the first point of the next
/// diagonal.
theorem nat_pair_zigzag_hit_next_diagonal(d: Nat) {
    nat_pair_zigzag_hits_pair(d, Nat.0) implies
        nat_pair_zigzag_hits_pair(Nat.0, d.suc)
} by {
    if nat_pair_zigzag_hits_pair(d, Nat.0) {
        let n: Nat satisfy {
            nat_pair_zigzag(n) = Pair.new(d, Nat.0)
        }
        nat_pair_next_zero_second(d)
        nat_pair_zigzag(n.suc) = nat_pair_next(nat_pair_zigzag(n))
        nat_pair_zigzag(n.suc) = nat_pair_next(Pair.new(d, Nat.0))
        nat_pair_zigzag(n.suc) = Pair.new(Nat.0, d.suc)
        exists(next_idx: Nat) {
            nat_pair_zigzag(next_idx) = Pair.new(Nat.0, d.suc)
        }
    }
}

/// Subtracting after a successor step exposes the previous predecessor on a
/// bounded diagonal.
theorem sub_suc_on_bounded_diagonal(d: Nat, a: Nat) {
    a.suc <= d implies d - a = (d - a.suc).suc
} by {
    if a.suc <= d {
        a <= d
        add_sub(d, a)
        d - a + a = d
        add_sub(d, a.suc)
        d - a.suc + a.suc = d
        (d - a.suc).suc + a = d - a.suc + a.suc
        (d - a.suc).suc + a = d
        add_cancels_right(a, d - a, (d - a.suc).suc)
        d - a = (d - a.suc).suc
    }
}

/// Starting from the first point of a diagonal, every bounded point on that
/// diagonal occurs.
theorem nat_pair_zigzag_diagonal_from_zero(d: Nat, a: Nat) {
    nat_pair_zigzag_hits_pair(Nat.0, d) and a <= d implies
        nat_pair_zigzag_hits_pair(a, d - a)
} by {
    define p(x: Nat) -> Bool {
        nat_pair_zigzag_hits_pair(Nat.0, d) and x <= d implies
            nat_pair_zigzag_hits_pair(x, d - x)
    }
    p(Nat.0)
    forall(x: Nat) {
        if p(x) {
            if nat_pair_zigzag_hits_pair(Nat.0, d) and x.suc <= d {
                p(x)
                nat_pair_zigzag_hits_pair(x, d - x)
                sub_suc_on_bounded_diagonal(d, x)
                d - x = (d - x.suc).suc
                nat_pair_zigzag_hits_pair(x, (d - x.suc).suc)
                nat_pair_zigzag_hit_step_down(x, d - x.suc)
                nat_pair_zigzag_hits_pair(x.suc, d - x.suc)
            }
            p(x.suc)
        }
    }
    p(a)
}

/// Predicate that the whole diagonal with coordinate sum `d` occurs.
define nat_pair_zigzag_hits_diagonal(d: Nat) -> Bool {
    forall(a: Nat) {
        a <= d implies nat_pair_zigzag_hits_pair(a, d - a)
    }
}

/// A diagonal occurrence predicate applies at each bounded coordinate.
theorem nat_pair_zigzag_hits_diagonal_at(d: Nat, a: Nat) {
    nat_pair_zigzag_hits_diagonal(d) and a <= d implies
        nat_pair_zigzag_hits_pair(a, d - a)
} by {
    if nat_pair_zigzag_hits_diagonal(d) and a <= d {
        nat_pair_zigzag_hits_diagonal(d) = forall(x: Nat) {
            x <= d implies nat_pair_zigzag_hits_pair(x, d - x)
        }
        nat_pair_zigzag_hits_pair(a, d - a)
    }
}

/// The zero diagonal occurs.
theorem nat_pair_zigzag_hits_diagonal_zero {
    nat_pair_zigzag_hits_diagonal(Nat.0)
} by {
    forall(a: Nat) {
        if a <= Nat.0 {
            a = Nat.0
            nat_pair_zigzag_hits_zero_zero
            Nat.0 - Nat.0 = Nat.0
            nat_pair_zigzag_hits_pair(a, Nat.0 - a)
        }
    }
}

/// If one diagonal occurs, then the next diagonal occurs.
theorem nat_pair_zigzag_hits_diagonal_step(d: Nat) {
    nat_pair_zigzag_hits_diagonal(d) implies nat_pair_zigzag_hits_diagonal(d.suc)
} by {
    if nat_pair_zigzag_hits_diagonal(d) {
        d <= d
        nat_pair_zigzag_hits_diagonal_at(d, d)
        nat_pair_zigzag_hits_pair(d, d - d)
        d - d = Nat.0
        nat_pair_zigzag_hits_pair_transport(d, d - d, d, Nat.0)
        nat_pair_zigzag_hits_pair(d, Nat.0)
        nat_pair_zigzag_hit_next_diagonal(d)
        nat_pair_zigzag_hits_pair(Nat.0, d.suc)
        forall(a: Nat) {
            if a <= d.suc {
                nat_pair_zigzag_diagonal_from_zero(d.suc, a)
                nat_pair_zigzag_hits_pair(a, d.suc - a)
            }
        }
        nat_pair_zigzag_hits_diagonal(d.suc)
    }
}

/// Every diagonal occurs in the diagonal enumeration.
theorem nat_pair_zigzag_hits_diagonal_all(d: Nat) {
    nat_pair_zigzag_hits_diagonal(d)
} by {
    nat_pair_zigzag_hits_diagonal_zero
    forall(k: Nat) {
        if nat_pair_zigzag_hits_diagonal(k) {
            nat_pair_zigzag_hits_diagonal_step(k)
            nat_pair_zigzag_hits_diagonal(k.suc)
        }
    }
}

/// Every pair of natural numbers occurs in the diagonal enumeration.
theorem nat_pair_zigzag_hits(a: Nat, b: Nat) {
    nat_pair_zigzag_hits_pair(a, b)
} by {
    let d = a + b
    nat_pair_zigzag_hits_diagonal_all(d)
    a <= d
    d - a = b
    nat_pair_zigzag_hits_diagonal_at(d, a)
    nat_pair_zigzag_hits_pair(a, d - a)
    nat_pair_zigzag_hits_pair_transport(a, d - a, a, b)
    nat_pair_zigzag_hits_pair(a, b)
}

/// The diagonal enumeration is surjective onto pairs of natural numbers.
theorem nat_pair_zigzag_surjective {
    is_surjective_fn(nat_pair_zigzag)
} by {
    forall(pair: Pair[Nat, Nat]) {
        nat_pair_zigzag_hits(pair.first, pair.second)
        let n: Nat satisfy {
            nat_pair_zigzag(n) = Pair.new(pair.first, pair.second)
        }
        pair_eta(pair)
        Pair.new(pair.first, pair.second) = pair
        exists(preimage: Nat) {
            nat_pair_zigzag(preimage) = pair
        }
    }
}

/// Every natural pair has an explicit preimage under the diagonal enumeration.
theorem nat_pair_zigzag_has_preimage(pair: Pair[Nat, Nat]) {
    exists(n: Nat) {
        nat_pair_zigzag(n) = pair
    }
} by {
    nat_pair_zigzag_surjective
    surjective_fn_has_preimage(nat_pair_zigzag, pair)
}

/// Convert a natural pair into an integer-natural pair using the zig-zag
/// enumeration on the first coordinate.
define int_nat_pair_of_nat_pair(pair: Pair[Nat, Nat]) -> Pair[Int, Nat] {
    Pair.new(int_zigzag(pair.first), pair.second)
}

/// The coordinate conversion unfolds on a newly formed pair.
theorem int_nat_pair_of_nat_pair_new(n: Nat, d: Nat) {
    int_nat_pair_of_nat_pair(Pair.new(n, d)) = Pair.new(int_zigzag(n), d)
} by {
    pair_new_first(n, d)
    pair_new_second(n, d)
}

/// The coordinate conversion is surjective onto integer-natural pairs.
theorem int_nat_pair_of_nat_pair_surjective {
    is_surjective_fn(int_nat_pair_of_nat_pair)
} by {
    forall(target: Pair[Int, Nat]) {
        int_zigzag_surjective
        surjective_fn_has_preimage(int_zigzag, target.first)
        let n: Nat satisfy {
            int_zigzag(n) = target.first
        }
        let preimage = Pair.new(n, target.second)
        int_nat_pair_of_nat_pair_new(n, target.second)
        int_nat_pair_of_nat_pair(preimage) = Pair.new(int_zigzag(n), target.second)
        pair_eta(target)
        Pair.new(target.first, target.second) = target
        int_nat_pair_of_nat_pair(preimage) = target
        exists(pair_preimage: Pair[Nat, Nat]) {
            int_nat_pair_of_nat_pair(pair_preimage) = target
        }
    }
}

/// The integer-natural pair enumeration obtained by composing the natural-pair
/// diagonal enumeration with the coordinate conversion.
define int_nat_pair_zigzag(n: Nat) -> Pair[Int, Nat] {
    int_nat_pair_of_nat_pair(nat_pair_zigzag(n))
}

/// The integer-natural pair enumeration is surjective.
theorem int_nat_pair_zigzag_surjective {
    is_surjective_fn(int_nat_pair_zigzag)
} by {
    forall(target: Pair[Int, Nat]) {
        int_nat_pair_of_nat_pair_surjective
        surjective_fn_has_preimage(int_nat_pair_of_nat_pair, target)
        let pair_preimage: Pair[Nat, Nat] satisfy {
            int_nat_pair_of_nat_pair(pair_preimage) = target
        }
        nat_pair_zigzag_surjective
        surjective_fn_has_preimage(nat_pair_zigzag, pair_preimage)
        let n: Nat satisfy {
            nat_pair_zigzag(n) = pair_preimage
        }
        int_nat_pair_zigzag(n) = int_nat_pair_of_nat_pair(nat_pair_zigzag(n))
        int_nat_pair_zigzag(n) = int_nat_pair_of_nat_pair(pair_preimage)
        int_nat_pair_zigzag(n) = target
        exists(preimage: Nat) {
            int_nat_pair_zigzag(preimage) = target
        }
    }
}

/// Every integer-natural pair has an explicit preimage under the combined enumeration.
theorem int_nat_pair_zigzag_has_preimage(target: Pair[Int, Nat]) {
    exists(n: Nat) {
        int_nat_pair_zigzag(n) = target
    }
} by {
    int_nat_pair_zigzag_surjective
    surjective_fn_has_preimage(int_nat_pair_zigzag, target)
}

/// A positive integer is a natural successor embedded in the integers.
theorem from_nat_positive_eq_suc(k: Nat) {
    Int.from_nat(k).is_positive implies exists(n: Nat) {
        Int.from_nat(k) = Int.from_nat(n.suc)
    }
} by {
    if Int.from_nat(k).is_positive {
        if k = Nat.0 {
            Int.from_nat(k) = Int.0
            zero_not_pos
            false
        } else {
            let n: Nat satisfy { n.suc = k }
            Int.from_nat(k) = Int.from_nat(n.suc)
            exists(result: Nat) {
                Int.from_nat(k) = Int.from_nat(result.suc)
            }
        }
    }
}

/// A negative integer is not positive.
theorem neg_suc_not_positive(k: Nat) {
    not Int.neg_suc(k).is_positive
} by {
    Int.neg_suc(k).is_negative
    if Int.neg_suc(k).is_positive {
        pos_is_not_neg(Int.neg_suc(k))
        false
    }
}

/// A positive integer is a natural successor embedded in the integers.
theorem positive_int_eq_from_nat_suc(a: Int) {
    a.is_positive implies exists(n: Nat) {
        a = Int.from_nat(n.suc)
    }
} by {
    if a.is_positive {
        match a {
            Int.from_nat(k) {
                from_nat_positive_eq_suc(k)
                let n: Nat satisfy {
                    Int.from_nat(k) = Int.from_nat(n.suc)
                }
                a = Int.from_nat(n.suc)
                exists(result: Nat) {
                    a = Int.from_nat(result.suc)
                }
            }
            Int.neg_suc(k) {
                neg_suc_not_positive(k)
                false
            }
        }
    }
}

/// The pair enumerator unfolds on a newly formed pair.
theorem rat_pair_enum_new(num: Int, denom_pred: Nat) {
    rat_pair_enum(Pair.new(num, denom_pred)) =
        reduce(num, Int.from_nat(denom_pred.suc))
} by {
    pair_new_first(num, denom_pred)
    pair_new_second(num, denom_pred)
}

/// Every rational number is represented by `rat_pair_enum`.
theorem rat_pair_enum_surjective {
    is_surjective_fn(rat_pair_enum)
} by {
    forall(r: Rat) {
        denom_positive(r)
        positive_int_eq_from_nat_suc(r.denom)
        let d: Nat satisfy {
            r.denom = Int.from_nat(d.suc)
        }
        let pair = Pair.new(r.num, d)
        rat_pair_enum_new(r.num, d)
        rat_pair_enum(pair) = reduce(r.num, Int.from_nat(d.suc))
        reduce_idempotent(r)
        reduce(r.num, r.denom) = r
        rat_pair_enum(pair) = r
        exists(preimage: Pair[Int, Nat]) {
            rat_pair_enum(preimage) = r
        }
    }
}

/// Every rational has an explicit integer-natural representation in `rat_pair_enum`.
theorem rat_pair_enum_has_preimage(r: Rat) {
    exists(pair: Pair[Int, Nat]) {
        rat_pair_enum(pair) = r
    }
} by {
    rat_pair_enum_surjective
    surjective_fn_has_preimage(rat_pair_enum, r)
}

/// A natural-number enumeration of the rationals.
define rat_zigzag(n: Nat) -> Rat {
    rat_pair_enum(int_nat_pair_zigzag(n))
}

/// The rational zig-zag enumeration is surjective.
theorem rat_zigzag_surjective {
    is_surjective_fn(rat_zigzag)
} by {
    forall(r: Rat) {
        rat_pair_enum_surjective
        surjective_fn_has_preimage(rat_pair_enum, r)
        let pair_preimage: Pair[Int, Nat] satisfy {
            rat_pair_enum(pair_preimage) = r
        }
        int_nat_pair_zigzag_surjective
        surjective_fn_has_preimage(int_nat_pair_zigzag, pair_preimage)
        let n: Nat satisfy {
            int_nat_pair_zigzag(n) = pair_preimage
        }
        rat_zigzag(n) = rat_pair_enum(int_nat_pair_zigzag(n))
        rat_zigzag(n) = rat_pair_enum(pair_preimage)
        rat_zigzag(n) = r
        exists(preimage: Nat) {
            rat_zigzag(preimage) = r
        }
    }
}

/// Every rational has an explicit preimage under the rational zig-zag enumeration.
theorem rat_zigzag_has_preimage(r: Rat) {
    exists(n: Nat) {
        rat_zigzag(n) = r
    }
} by {
    rat_zigzag_surjective
    surjective_fn_has_preimage(rat_zigzag, r)
}


/// The rational numbers admit an enumeration by natural numbers.
theorem rational_numbers_are_denumerable {
    exists(enumeration: Nat -> Rat) {
        is_surjective_fn(enumeration)
    }
} by {
    rat_zigzag_surjective
    exists(enumeration: Nat -> Rat) {
        is_surjective_fn(enumeration)
    }
}

/// The natural-number embedding into the integers is injective.
theorem int_from_nat_injective(a: Nat, b: Nat) {
    Int.from_nat(a) = Int.from_nat(b) implies a = b
}

/// The natural-number embedding into the rationals is injective.
theorem rat_from_nat_injective {
    is_injective_fn(Rat.from_nat)
} by {
    forall(a: Nat, b: Nat) {
        if Rat.from_nat(a) = Rat.from_nat(b) {
            Rat.from_nat(a) = Rat.from_int(Int.from_nat(a))
            Rat.from_nat(b) = Rat.from_int(Int.from_nat(b))
            Rat.from_int(Int.from_nat(a)) = Rat.from_int(Int.from_nat(b))
            from_int_cancel(Int.from_nat(a), Int.from_nat(b))
            Int.from_nat(a) = Int.from_nat(b)
            int_from_nat_injective(a, b)
            a = b
        }
    }
}

/// The chosen first-preimage map for the rational enumeration is an injective
/// encoding of rationals by natural numbers.
theorem rat_zigzag_inverse_injective {
    is_injective_fn(inverse_fn(rat_zigzag))
} by {
    rat_zigzag_surjective
    forall(r1: Rat, r2: Rat) {
        if inverse_fn(rat_zigzag, r1) = inverse_fn(rat_zigzag, r2) {
            inverse_fn_apply_of_surjective(rat_zigzag, r1)
            inverse_fn_apply_of_surjective(rat_zigzag, r2)
            rat_zigzag(inverse_fn(rat_zigzag, r1)) = r1
            rat_zigzag(inverse_fn(rat_zigzag, r2)) = r2
            r1 = r2
        }
    }
}

/// The rational numbers have a natural-number enumeration and contain an
/// embedded copy of the natural numbers.
theorem rational_numbers_have_countably_infinite_enumeration {
    exists(enumeration: Nat -> Rat) {
        is_surjective_fn(enumeration)
    } and exists(encoding: Rat -> Nat) {
        is_injective_fn(encoding)
    } and exists(embedding: Nat -> Rat) {
        is_injective_fn(embedding)
    }
} by {
    rational_numbers_are_denumerable
    exists(enumeration: Nat -> Rat) {
        is_surjective_fn(enumeration)
    }
    rat_zigzag_inverse_injective
    exists(encoding: Rat -> Nat) {
        is_injective_fn(encoding)
    }
    rat_from_nat_injective
    exists(embedding: Nat -> Rat) {
        is_injective_fn(embedding)
    }
}
