from nat import Nat, add_one_right, alt_suc_ne_zero
from int import Int, abs, unit_sign, pos_part_from, div_pos_imp_lte, mul_pos_pos
from rat.rat_base import Rat, reduce, rat_is_reflexive, reduce_idempotent, reduce_cancels_right, lte_from_int,
    from_int_num, from_int_denom, mul_reduced_int_right, lt_add_right, mul_zero_right,
    neg_imp_lt_zero, from_int_cancel, sub_from_int, unreduce_right
from order import LinearOrder, min_lte_left, min_lte_right, lte_min_of_bounds, lte_max_left,
    lte_max_right, max_lte_of_upper_bounds, min_max_distrib_left, max_min_distrib_left

// It will be convenient to have a particular function that
// approaches zero.
define iop(n: Nat) -> Rat {
    Rat.1 / (Rat.1 + Rat.from_nat(n))
}

theorem iop_pos(n: Nat) {
    iop(n).is_positive
} by {
    (Rat.1 + Rat.from_nat(n)).is_positive
    (Rat.1 + Rat.from_nat(n)).inverse.is_positive
    Rat.1 * (Rat.1 + Rat.from_nat(n)).inverse = (Rat.1 + Rat.from_nat(n)).inverse
    Rat.1 * (Rat.1 + Rat.from_nat(n)).inverse = Rat.1 / (Rat.1 + Rat.from_nat(n))
    Rat.1 / (Rat.1 + Rat.from_nat(n)) = iop(n)
}

theorem pos_ne_zero(a: Rat) {
    a.is_positive implies a != Rat.0
}

theorem iop_ne_zero(n: Nat) {
    iop(n) != Rat.0
}

theorem iop_recip(n: Nat) {
    iop(n).inverse = Rat.1 + Rat.from_nat(n)
} by {
    (Rat.1 + Rat.from_nat(n)).is_positive
    (Rat.1 + Rat.from_nat(n)).inverse = iop(n)
}

theorem iop_mul_lt_one(n: Nat) {
    iop(n) * Rat.from_nat(n) < Rat.1
} by {
    iop(n) + iop(n) * Rat.from_nat(n) = Rat.1
    iop(n).is_positive
    iop(n) * Rat.from_nat(n) + iop(n) = iop(n) + iop(n) * Rat.from_nat(n)
    not iop(n).is_positive or iop(n) * Rat.from_nat(n) < iop(n) * Rat.from_nat(n) + iop(n)
}

theorem pos_lte_num(a: Rat) {
    a.is_positive implies a <= Rat.from_int(a.num)
} by {
    a.num <= a.denom * a.num
}

theorem lt_some_int(a: Rat) {
    exists(n: Int) {
        a < Rat.from_int(n)
    }
}

theorem lt_some_nat(a: Rat) {
    exists(n: Nat) {
        a < Rat.from_nat(n)
    }
} by {
    let i: Int satisfy {
        a < Rat.from_int(i)
    }
    let n = abs(i)
    i <= Int.from_nat(abs(i))
    Rat.from_int(i) <= Rat.from_int(Int.from_nat(n))
    Rat.from_int(Int.from_nat(n)) = Rat.from_nat(n)
    Rat.from_int(i) <= Rat.from_nat(n)
}

theorem iop_gets_lt(eps: Rat) {
    eps.is_positive implies exists(n: Nat) {
        forall(i: Nat) {
            n <= i implies iop(i) < eps
        }
    }
} by {
    let n: Nat satisfy {
        eps.inverse < Rat.from_nat(n)
    }
    forall(i: Nat) {
        if n <= i {
            Rat.from_nat(n) <= Rat.from_nat(i)
            eps.inverse * eps < Rat.from_nat(i) * eps
            Rat.1 < Rat.from_nat(i) * eps
            Rat.from_nat(i) * iop(i) < Rat.from_nat(i) * eps
            iop(i) < eps
        }
    }
}

theorem three_is_positive {
    Rat.3.is_positive
}

theorem times_three(x: Rat) {
    Rat.3 * x = x + x + x
} by {
    Int.3 = Int.2 + Int.1
    Rat.2 * x + Rat.1 * x = (Rat.2 + Rat.1) * x
    Rat.from_int(Int.2) + Rat.from_int(Int.1) = Rat.from_int(Int.2 + Int.1)
    Rat.2 * x = x + x
    Rat.1 * x = x
}

theorem three_thirds(x: Rat) {
    (x / Rat.3) + (x / Rat.3) + (x / Rat.3) = x
}

theorem some_mul_lt(a: Rat, b: Rat) {
    Rat.0 <= a and b.is_positive implies
    exists(eps: Rat) {
        eps.is_positive and a * eps < b
    }
} by {
    if a = Rat.0 {
        let eps = b
        eps.is_positive and a * eps < b
    } else {
        a.is_positive
        let c: Rat satisfy {
            c.is_positive and c < b
        }
        let eps = c / a
        a * eps < b
        eps.is_positive and a * eps < b
    }
}

theorem close_mul_pos(a: Rat, b: Rat, eps: Rat, r: Rat) {
    r.is_positive and a.is_close(b, eps)
    implies
    (a * r).is_close(b * r, eps * r)
} by {
    a > b - eps
    (b - eps) * r = r * (b - eps)
    r * b - r * eps = r * (b - eps)
    (b - eps) * r < a * r
    b * r = r * b
    eps * r = r * eps
    a * r > b * r - eps * r
    a * r < (b * r) + (eps * r)
}

theorem close_neg(a: Rat, b: Rat, eps: Rat) {
    a.is_close(b, eps) implies (-a).is_close(-b, eps)
} by {
    (-a - -b).abs < eps
}

theorem lte_mul_nonneg(a: Rat, b: Rat, c: Rat) {
    a <= b and Rat.0 <= c implies a * c <= b * c
} by {
    if c = Rat.0 {
    } else {
        a * c <= b * c
    }
}

theorem bounding_both(a: Rat, b: Rat) {
    exists(c: Rat) {
        a < c and b < c
    }
} by {
    a < b or a = b or a > b
    if a < b {
        let c: Rat satisfy {
            b < c
        }
        a < c
        a < c and b < c
    } else {
        let c: Rat satisfy {
            a < c
        }
        if a = b {
            b < c
            a < c and b < c
        } else {
            a > b
            a > b = b < a
            b < a
            b < c
            a < c and b < c
        }
    }
}

theorem bound_yields_finite_seq_abs_bounded(a: Nat -> Rat, n: Nat, bound: Rat) {
    forall(i: Nat) {
        i <= n implies a(i).abs < bound
    }
    implies
    exists(witness: Rat) {
        forall(i: Nat) {
            i <= n implies a(i).abs < witness
        }
    }
} by {
    if forall(i: Nat) {
        i <= n implies a(i).abs < bound
    } {
        forall(i: Nat) {
            i <= n implies a(i).abs < bound
        }
    }
}

theorem finite_seq_abs_bounded(a: Nat -> Rat, n: Nat) {
    exists(bound: Rat) {
        forall(i: Nat) {
            i <= n implies a(i).abs < bound
        }
    }
} by {
    define p(k: Nat) -> Bool {
        exists(bound: Rat) {
            forall(i: Nat) {
                i <= k implies a(i).abs < bound
            }
        }
    }
    let zero_bound: Rat satisfy {
        a(Nat.0).abs < zero_bound
    }
    forall(i: Nat) {
        i <= Nat.0 implies a(i).abs < zero_bound
    }
    exists(bound: Rat) {
        forall(i: Nat) {
            i <= Nat.0 implies a(i).abs < bound
        }
    }
    p(Nat.0)
    forall(k: Nat) {
        if p(k) {
            let base_bound: Rat satisfy {
                forall(i: Nat) {
                    i <= k implies a(i).abs < base_bound
                }
            }
            let extra_bound: Rat satisfy {
                a(k.suc).abs < extra_bound
            }
            let bound: Rat satisfy {
                base_bound < bound and extra_bound < bound
            }
            forall(i: Nat) {
                if i <= k.suc {
                    if i <= k {
                        a(i).abs < bound
                    } else {
                        i = k.suc
                        a(i).abs < bound
                    }
                }
            }
            forall(i: Nat) {
                i <= k.suc implies a(i).abs < bound
            }
            let witness_bound = bound
            forall(i: Nat) {
                i <= k.suc implies a(i).abs < witness_bound
            }
            exists(next_bound: Rat) {
                forall(i: Nat) {
                    i <= k.suc implies a(i).abs < next_bound
                }
            }
            p(k.suc)
        }
    }
    p(n)
}

theorem abs_reduce_left(a: Int, b: Int) {
    reduce(a.abs, b).abs = reduce(a, b).abs
} by {
    Int.from_nat(abs(a)) = a.abs
    unit_sign(a) * Int.from_nat(abs(a)) = a
    unit_sign(a) * Int.from_nat(abs(a)) = Int.from_nat(abs(a)) * unit_sign(a)
    reduce(Int.from_nat(abs(a)), b) * Rat.from_int(unit_sign(a)) = reduce(Int.from_nat(abs(a)) * unit_sign(a), b)
    if a.is_negative {
        -Int.1 = unit_sign(a)
        reduce(a.abs, b) * -Rat.1 = -reduce(a.abs, b)
        (-reduce(a.abs, b)).abs = reduce(a.abs, b).abs
    } else {
        a.abs = a
    }
}

theorem reduce_neg_num(a: Int, b: Int) {
    reduce(-a, b) = -reduce(a, b)
}

theorem reduce_neg_denom(a: Int, b: Int) {
    reduce(a, -b) = -reduce(a, b)
} by {
    reduce(a, -b) = reduce(-Int.1 * a, -Int.1 * -b)
}

theorem abs_reduce_right(a: Int, b: Int) {
    reduce(a, b.abs).abs = reduce(a, b).abs
} by {
    if b.is_negative {
    } else {
    }
}

theorem reduce_nonneg(a: Int, b: Int) {
    not a.is_negative and not b.is_negative
    implies
    not reduce(a, b).is_negative
} by {
    if a = Int.0 {
    } else {
        if b = Int.0 {
            not reduce(a, b).is_negative
        } else {
            not reduce(a, b).is_negative
        }
    }
}

/// A positive reduction has numerator bounded by the unreduced numerator.
theorem reduce_num_lte_raw_of_positive(a: Int, b: Int) {
    reduce(a, b).is_positive and b.is_positive implies reduce(a, b).num <= a
} by {
    if reduce(a, b).is_positive and b.is_positive {
        b != Int.0
        let d: Int satisfy {
            reduce(a, b).num * d = a and reduce(a, b).denom * d = b
        }
        reduce(a, b).denom.is_positive
        reduce(a, b).denom * d = b
        (reduce(a, b).denom * d).is_positive
        not (reduce(a, b).denom * d).is_positive or d.is_positive or
            reduce(a, b).denom.is_negative
        not reduce(a, b).denom.is_negative
        d.is_positive
        reduce(a, b).num.is_positive
        mul_pos_pos(reduce(a, b).num, d)
        (reduce(a, b).num * d).is_positive
        a.is_positive
        d * reduce(a, b).num = a
        reduce(a, b).num.divides(a)
        div_pos_imp_lte(reduce(a, b).num, a)
        reduce(a, b).num <= a
    }
}

theorem reduce_abs(a: Int, b: Int) {
    reduce(a.abs, b.abs) = reduce(a, b).abs
} by {
    not a.abs.is_negative
    not b.abs.is_negative
    not reduce(a.abs, b.abs).is_negative
}

theorem lte_cancel_mul_pos(p: Rat, q: Rat, r: Rat) {
    r.is_positive and p * r <= q * r
    implies
    p <= q
} by {
    if p * r = q * r {
    } else {
        p <= q
    }
}

theorem reduce_lte(a: Int, b: Int, c: Int) {
    not c.is_negative and a <= b implies
    reduce(a, c) <= reduce(b, c)
} by {
    if c = Int.0 {
        reduce(a, c) = Rat.0
        reduce(b, c) = Rat.0
        reduce(a, c) = reduce(b, c)
        forall(x: Rat) {
            x <= x
        }
        reduce(a, c) <= reduce(a, c)
        reduce(a, c) <= reduce(b, c)
    } else {
        Rat.from_int(c).is_positive
        Rat.from_int(a) <= Rat.from_int(b)
        Rat.from_int(a).num = a
        Rat.from_int(a).denom = Int.1
        reduce(a, Int.1) = Rat.from_int(a)
        Rat.from_int(b).num = b
        Rat.from_int(b).denom = Int.1
        reduce(b, Int.1) = Rat.from_int(b)
        reduce(a, c) * Rat.from_int(c) = reduce(a * c, c)
        reduce(b, c) * Rat.from_int(c) = reduce(b * c, c)
        reduce(a, c) * Rat.from_int(c) = Rat.from_int(a)
        reduce(b, c) * Rat.from_int(c) = Rat.from_int(b)
        reduce(a, c) * Rat.from_int(c) <= reduce(b, c) * Rat.from_int(c)
        reduce(a, c) <= reduce(b, c)
    }
}

theorem triangle_ineq(a: Rat, b: Rat) {
    (a + b).abs <= a.abs + b.abs
} by {
    let (an: Int, bn: Int, c: Int) satisfy {
        a = reduce(an, c) and b = reduce(bn, c)
    }
    (an + bn).abs <= an.abs + bn.abs
    not c.abs.is_negative
    reduce((an + bn).abs, c.abs) <= reduce(an.abs + bn.abs, c.abs)
}

// Not all that generally useful.
theorem diff_mul_bound(a0: Rat, b0: Rat, a1: Rat, b1: Rat) {
    (a0 * b0 - a1 * b1).abs <= a0.abs * (b0 - b1).abs + b1.abs * (a0 - a1).abs
} by {
    a0 * (b0 - b1) + b1 * (a0 - a1) = a0 * b0 - a0 * b1 + b1 * a0 - b1 * a1
    (a0 * (b0 - b1) + b1 * (a0 - a1)).abs <= (a0 * (b0 - b1)).abs + (b1 * (a0 - a1)).abs
}

theorem nonneg_lt_imp_pos(a: Rat, b: Rat) {
    not a.is_negative and a < b implies
    b.is_positive
} by {
    Rat.0 <= a
    Rat.0 < b
}

theorem lt_pos_mul_lt_pos(a: Rat, b: Rat, c: Rat, d: Rat) {
    not a.is_negative and not c.is_negative and a < b and c < d
    implies
    a * c < b * d
} by {
    a * c <= b * c
}

theorem abs_nonneg(a: Rat) {
    not a.abs.is_negative
}

theorem lte_mul_lte(a: Rat, b: Rat, c: Rat, d: Rat) {
    not a.is_negative and not c.is_negative and a <= b and c <= d
    implies
    a * c <= b * d
} by {
    if a = b {
        c * b <= d * b
    } else {
        if c = d {
            a * c <= b * d
        } else {
            a * c < b * d
            a * c <= b * d
        }
    }
}

theorem add_div_distrib(a: Rat, b: Rat, c: Rat) {
    (a + b) / c = a / c + b / c
}

theorem sub_div_distrib(a: Rat, b: Rat, c: Rat) {
    (a - b) / c = a / c - b / c
} by {
    c.inverse * a - c.inverse * b = c.inverse * (a - b)
    (a - b) * c.inverse = (a - b) / c
    a * c.inverse = a / c
    b * c.inverse = b / c
    (a - b) * c.inverse = c.inverse * (a - b)
    a * c.inverse = c.inverse * a
    b * c.inverse = c.inverse * b
}

theorem recip_mul(a: Rat, b: Rat) {
    (a * b).inverse = a.inverse * b.inverse
} by {
    if a = Rat.0 {
        // Degenerate but it works
        a * b = Rat.0
        (a * b).inverse = Rat.0
    } else {
        if b = Rat.0 {
            (a * b).inverse = a.inverse * b.inverse
        } else {
            (a * b).inverse * (a * b) = Rat.1
            (a * b).inverse = a.inverse * b.inverse
        }
    }
}

theorem mul_fractions(a: Rat, b: Rat, c: Rat, d: Rat) {
    (a / b) * (c / d) = (a * c) / (b * d)
} by {
    (a / b) * (c / d) = (a * c) * (b.inverse * d.inverse)
}

theorem cancel_left_num_denom(a: Rat, b: Rat, c: Rat) {
    a != Rat.0 implies
    (a * b) / (a * c) = b / c
}

theorem cancel_to_inverse(a: Rat, b: Rat) {
    a != Rat.0 and b != Rat.0 implies
    a / (a * b) = b.inverse
}

theorem recip_diff(a: Rat, b: Rat) {
    a != Rat.0 and b != Rat.0 implies
    a.inverse - b.inverse = (b - a) / (a * b)
}
theorem lt_mul_lte(a: Rat, b: Rat, c: Rat, d: Rat) {
    not a.is_negative and c.is_positive and a < b and c <= d
    implies
    a * c < b * d
}

// This isn't ideal but this is how we have defined division.
theorem div_zero(a: Rat) {
    a / Rat.0 = Rat.0
} by {
    Rat.0.num = Int.0
    reduce(Rat.0.denom, Int.0) = Rat.0
    reduce(Rat.0.denom, Rat.0.num) = Rat.0.inverse
    Rat.0.inverse = Rat.0
    a * Rat.0.inverse = a / Rat.0
}

theorem recip_recip(a: Rat) {
    a.inverse.inverse = a
}

theorem neg_recip(a: Rat) {
    a.is_negative implies a.inverse.is_negative
} by {
    if a.inverse = Rat.0 {
        reduce(Rat.0.denom, Rat.0.num) = Rat.0.inverse
        Int.0 != Int.0 or reduce(Rat.0.denom, Int.0) = Rat.0
        not a.inverse.is_negative or a.inverse < Rat.0
        a.inverse.inverse = a
        not a.inverse < Rat.0 or a.inverse != Rat.0
        false
    }
}

theorem div_neg_neg(a: Rat, b: Rat) {
    a.is_negative and b.is_negative
    implies
    (a / b).is_positive
}

theorem div_negs_cancel(a: Rat, b: Rat) {
    a / b = (-a) / (-b)
} by {
    -Rat.1 * a = -a
    -Rat.1 * b = -b
    -Rat.1 != Rat.0
    -Rat.1 * a / (-Rat.1 * b) = a / b
}

theorem abs_div(a: Rat, b: Rat) {
    b != Rat.0 implies
    (a / b).abs = a.abs / b.abs
} by {
    if a.is_negative {
        if b.is_negative {
            a.abs = -a
            b.abs = -b
            (a / b).is_positive
            (a / b).abs = a / b
            -a / -b = a / b
            (a / b).abs = a.abs / b.abs
        } else {
            (a / b).abs = -(a / b)
            (a / b).abs = a.abs / b.abs
        }
    } else {
        if b.is_negative {
            (a / b).abs = -(a / b)
            (a / b).abs = a.abs / b.abs
        } else {
            not (a / b).is_negative
            (a / b).abs = a.abs / b.abs
        }
    }
}

theorem lt_make_left_denom(a: Rat, b: Rat, c: Rat) {
    c.is_positive and a < b * c implies a / c < b
}

theorem lt_make_right_denom(a: Rat, b: Rat, c: Rat) {
    b.is_positive and a * b < c implies a < c / b
} by {
    a * b * b.inverse = a
}

theorem lt_elim_left_denom(a: Rat, b: Rat, c: Rat) {
    b.is_positive and a / b < c implies a < c * b
}

/// Cross-multiplication preserves strict inequality for fractions with positive denominators.
theorem cross_mul_lt(a: Rat, b: Rat, c: Rat, d: Rat) {
    b.is_positive and d.is_positive and a * d < c * b
    implies
    a / b < c / d
} by {
    (b * d).is_positive
    a * d / (b * d) < c * b / (b * d)
    b != Rat.0
    d != Rat.0
    (a * d) / (b * d) = a / b
    (c * b) / (b * d) = c / d
}

/// Cross-multiplication preserves non-strict inequality for fractions with positive denominators.
theorem cross_mul_lte(a: Rat, b: Rat, c: Rat, d: Rat) {
    b.is_positive and d.is_positive and a * d <= c * b
    implies
    a / b <= c / d
} by {
    if a * d = c * b {
        (b * d).is_positive
        b != Rat.0
        d != Rat.0
        (a * d) / (b * d) = a / b
        (c * b) / (b * d) = c / d
        a / b = c / d
        a / b != c / d or a / b <= c / d
        a / b <= c / d
    } else {
        a * d < c * b
        a / b < c / d
        a / b < c / d = (a / b <= c / d and a / b != c / d)
        a / b <= c / d
    }
}

theorem abs_of_div(a: Rat, b: Rat) {
    (a / b).abs = a.abs / b.abs
} by {
    if b = Rat.0 {
        a / Rat.0 = Rat.0
        (a / Rat.0).abs = Rat.0
        Rat.0.abs = Rat.0
        b.abs = Rat.0
        a.abs / Rat.0 = Rat.0
        (a / b).abs = a.abs / b.abs
    } else {
        (a / b).abs = a.abs / b.abs
    }
}

theorem mul_div_swap(a: Rat, b: Rat, c: Rat) {
    a * b / c = (a / c) * b
}

theorem zero_recip(a: Rat) {
    Rat.0.inverse = Rat.0
}

// The multiplicative algebraic structure.

from algebra.semigroup import Semigroup

instance Rat: Semigroup

from algebra.monoid.monoid import Monoid

instance Rat: Monoid

from semiring import Semiring

instance Rat: Semiring

from algebra.ring.ring import Ring

instance Rat: Ring

from algebra.comm_semigroup import CommSemigroup

instance Rat: CommSemigroup

from algebra.comm_monoid import CommMonoid

instance Rat: CommMonoid

from comm_ring import CommRing

instance Rat: CommRing

from algebra.field.field import Field

theorem zero_is_different_than_one {
    Rat.0 != Rat.1
}

instance Rat: Field

theorem rat_total(a: Rat, b: Rat) { a <= b or b <= a } by {
    if a < b {
        a <= b
    } else {
        if a = b {
            a <= b
        } else {
            a > b
            a > b = b < a
            b <= a
        }
    }
}

theorem lte_add_right(a: Rat, b: Rat, c: Rat) {
    a <= b implies a + c <= b + c
} by {
    if a = b {
        a + c = b + c
    } else {
        a < b
        a + c <= b + c
    }
}

theorem lte_add_left(a: Rat, b: Rat, c: Rat) {
    b <= c implies a + b <= a + c
} by {
    a + b = b + a
    a + c = c + a
    b + a <= c + a
}

instance Rat: LinearOrder

from lattice import Meet, Join, MeetSemilattice, JoinSemilattice, Lattice, DistribLattice

instance Rat: Meet {
    define meet(self, other: Rat) -> Rat {
        self.min(other)
    }
}

instance Rat: Join {
    define join(self, other: Rat) -> Rat {
        self.max(other)
    }
}

theorem rat_meet_lte_left(a: Rat, b: Rat) {
    a.meet(b) <= a
} by {
    min_lte_left(a, b)
}

theorem rat_meet_lte_right(a: Rat, b: Rat) {
    a.meet(b) <= b
} by {
    min_lte_right(a, b)
}

theorem rat_lte_meet_of_bounds(c: Rat, a: Rat, b: Rat) {
    c <= a and c <= b implies c <= a.meet(b)
} by {
    if c <= a and c <= b {
        lte_min_of_bounds(c, a, b)
        c <= a.meet(b)
    }
}

instance Rat: MeetSemilattice

theorem rat_lte_join_left(a: Rat, b: Rat) {
    a <= a.join(b)
} by {
    lte_max_left(a, b)
}

theorem rat_lte_join_right(a: Rat, b: Rat) {
    b <= a.join(b)
} by {
    lte_max_right(a, b)
}

theorem rat_join_lte_of_bounds(a: Rat, b: Rat, c: Rat) {
    a <= c and b <= c implies a.join(b) <= c
} by {
    if a <= c and b <= c {
        max_lte_of_upper_bounds(a, b, c)
        a.join(b) <= c
    }
}

instance Rat: JoinSemilattice

instance Rat: Lattice

theorem rat_meet_join_distrib_left(a: Rat, b: Rat, c: Rat) {
    a.meet(b.join(c)) = a.meet(b).join(a.meet(c))
} by {
    min_max_distrib_left(a, b, c)
}

theorem rat_join_meet_distrib_left(a: Rat, b: Rat, c: Rat) {
    a.join(b.meet(c)) = a.join(b).meet(a.join(c))
} by {
    max_min_distrib_left(a, b, c)
}

instance Rat: DistribLattice

from algebra.add_ordered_group import AddLeftOrderedGroup, AddOrderedGroup

instance Rat: AddLeftOrderedGroup
instance Rat: AddOrderedGroup

from ordered_field import OrderedField

theorem mul_nonnegative(a: Rat, b: Rat) {
    Rat.0 <= a and Rat.0 <= b implies Rat.0 <= a * b
} by {
    Rat.0 * Rat.0 <= a * b
    Rat.0 * Rat.0 = Rat.0
}

theorem from_int_lte_cancel(a: Int, b: Int) {
    Rat.from_int(a) <= Rat.from_int(b) implies a <= b
} by {
    if Rat.from_int(a) <= Rat.from_int(b) {
        Rat.from_int(a) <= Rat.from_int(b) =
            ((Rat.from_int(b) - Rat.from_int(a)).is_positive or
                Rat.from_int(a) = Rat.from_int(b))
        if (Rat.from_int(b) - Rat.from_int(a)).is_positive {
            sub_from_int(b, a)
            Rat.from_int(b) - Rat.from_int(a) = Rat.from_int(b - a)
            Rat.from_int(b - a).num = b - a
            Rat.from_int(b - a).is_positive =
                Rat.from_int(b - a).num.is_positive
            (b - a).is_positive
            a <= b = ((b - a).is_positive or a = b)
            a <= b
        } else {
            Rat.from_int(a) = Rat.from_int(b)
            from_int_cancel(a, b)
            a = b
            a <= b
        }
    }
}

theorem from_int_lt_cancel(a: Int, b: Int) {
    Rat.from_int(a) < Rat.from_int(b) implies a < b
} by {
    if Rat.from_int(a) < Rat.from_int(b) {
        Rat.from_int(a) < Rat.from_int(b) =
            (Rat.from_int(a) <= Rat.from_int(b) and
                Rat.from_int(a) != Rat.from_int(b))
        Rat.from_int(a) <= Rat.from_int(b)
        from_int_lte_cancel(a, b)
        a <= b
        if a = b {
            Rat.from_int(a) = Rat.from_int(b)
            false
        }
        a != b
        a < b = (a <= b and a != b)
        a < b
    }
}

instance Rat: OrderedField
