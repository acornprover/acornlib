/// Public interface for rational numbers.

from algebra.add import Add
from algebra.inverse import Inverse
from lte import LTE
from algebra.mul import Mul
from nat import Nat
from algebra.neg import Neg
from int import Int, unit_sign, abs, pos_is_not_neg, zero_not_pos
from algebra.zero import Zero
from algebra.one import One
from algebra.add_semigroup import AddSemigroup
from algebra.add_comm_semigroup import AddCommSemigroup
from algebra.add_monoid import AddMonoid
from algebra.add_comm_monoid import AddCommMonoid
from algebra.add_group import AddGroup
from algebra.add_comm_group import AddCommGroup
from data.basic.relation_basic import is_reflexive, is_transitive, is_antisymmetric, transitive_step
from order import PartialOrder
from order_relation import lt_relation_is_transitive
from order import LinearOrder, min_lte_left, min_lte_right, lte_min_of_bounds, lte_max_left,
    lte_max_right, max_lte_of_upper_bounds, min_max_distrib_left, max_min_distrib_left
from algebra.semigroup import Semigroup
from algebra.monoid.monoid import Monoid
from semiring import Semiring
from algebra.ring.ring import Ring
from algebra.comm_semigroup import CommSemigroup
from algebra.comm_monoid import CommMonoid
from comm_ring import CommRing
from algebra.field.field import Field
from lattice import Meet, Join, MeetSemilattice, JoinSemilattice, Lattice, DistribLattice
from algebra.add_ordered_group import AddLeftOrderedGroup, AddOrderedGroup
from ordered_field import OrderedField
from nat import add_cancels_right, add_sub
from pair import Pair, pair_eta, pair_new_first, pair_new_second
from data.basic.functions import inverse_fn, inverse_fn_apply_of_surjective, is_injective_fn,
    is_surjective_fn, surjective_fn_has_preimage

numerals Nat
numerals Int

// rat_base.ac
/// True if a fraction a/b is in reduced form (lowest terms).
/// The denominator must be positive and gcd(a,b) must be 1.
define is_reduced(a: Int, b: Int) -> Bool {
    b > Int.0 and a.gcd(b) = Int.1
}

theorem denom_one_is_reduced(a: Int) {
    is_reduced(a, Int.1)
}

/// Rational numbers represented as fractions in reduced form.
/// The constraint ensures the fraction is always in lowest terms with positive denominator.
structure Rat {
    /// The numerator of the rational number.
    num: Int
    /// The denominator of the rational number (always positive).
    denom: Int
} constraint {
    is_reduced(num, denom)
}

let Option.some(rat_zero) = Rat.new(Int.0, Int.1)

theorem denom_nonzero(r: Rat) {
    r.denom != Int.0
}

theorem denom_positive(r: Rat) {
    r.denom.is_positive
}

let rat_from_int(n: Int) -> r: Rat satisfy {
    Option.some(r) = Rat.new(n, Int.1)
}

theorem constraint_yields_new(a: Int, b: Int) {
    Rat.constraint(a, b) implies exists(r: Rat) {
        Option.some(r) = Rat.new(a, b)
    }
}

attributes Rat {
    /// Converts an integer to a rational number.
    let from_int = rat_from_int
}

instance Rat: Zero {
    let 0: Rat = rat_zero
}

theorem zero_num {
    Rat.0.num = Int.0
}

theorem zero_denom {
    Rat.0.denom = Int.1
}

theorem from_int_zero {
    Rat.from_int(Int.0) = Rat.0
}

theorem from_int_num(n: Int) {
    Rat.from_int(n).num = n
}

theorem from_int_denom(n: Int) {
    Rat.from_int(n).denom = Int.1
}

/// True if two fractions a/b and c/d are equal using cross multiplication.
/// Requires non-zero denominators to avoid the 0/0 edge case.
define cross_equals(a: Int, b: Int, c: Int, d: Int) -> Bool {
    b != Int.0 and d != Int.0 and a * d = c * b
}

theorem cross_equals_trans(a: Int, b: Int, c: Int, d: Int, e: Int, f: Int) {
    cross_equals(a, b, c, d) and cross_equals(c, d, e, f) implies cross_equals(a, b, e, f)
}

/// Reduces a fraction a/b to its simplest form as a Rat.
/// Yields 0 if b = 0 (treating division by zero as zero).
let reduce(a: Int, b: Int) -> r: Rat satisfy {
    if b = Int.0 {
        r = Rat.0
    } else {
        cross_equals(r.num, r.denom, a, b)
    }
}

theorem cross_eq_imp_eq(r1: Rat, r2: Rat) {
    cross_equals(r1.num, r1.denom, r2.num, r2.denom) implies r1 = r2
}

attributes Rat {
    /// The rational two.
    let 2: Rat = Rat.from_int(Int.2)
    /// The rational three.
    let 3: Rat = Rat.from_int(Int.3)
    /// The rational four.
    let 4: Rat = Rat.from_int(Int.4)
    /// The rational five.
    let 5: Rat = Rat.from_int(Int.5)
    /// The rational six.
    let 6: Rat = Rat.from_int(Int.6)
    /// The rational seven.
    let 7: Rat = Rat.from_int(Int.7)
    /// The rational eight.
    let 8: Rat = Rat.from_int(Int.8)
    /// The rational nine.
    let 9: Rat = Rat.from_int(Int.9)
    /// The rational ten.
    let 10: Rat = Rat.from_int(Int.10)

    /// True if the rational is positive.
    define is_positive(self) -> Bool {
        self.num.is_positive
    }

    /// True if the rational is negative.
    define is_negative(self) -> Bool {
        self.num.is_negative
    }
}


instance Rat: One {
    let 1: Rat = Rat.from_int(Int.1)
}

let neg(a: Rat) -> b: Rat satisfy {
    Rat.new(-a.num, a.denom) = Option.some(b)
}

/// The negation of a rational number.
instance Rat: Neg {
    let neg = neg
}

/// The sum of two rational numbers.
instance Rat: Add {
    define add(self, other: Rat) -> Rat {
        reduce(self.num * other.denom + other.num * self.denom,
               self.denom * other.denom)
    }
}

/// The product of two rational numbers.
instance Rat: Mul {
    define mul(self, other: Rat) -> Rat {
        reduce(self.num * other.num, self.denom * other.denom)
    }
}

/// The inverse of a rational number (1/x).
/// The inverse of zero is defined to be zero.
instance Rat: Inverse {
    define inverse(self) -> Rat {
        reduce(self.denom, self.num)
    }
}

attributes Rat {
    /// The quotient of two rational numbers.
    /// Division by zero is defined to yield zero.
    define div(self, other: Rat) -> Rat {
        self * other.inverse
    }

    /// The rational formed by appending a digit to this rational in base 10.
    define read(self, other: Rat) -> Rat { Rat.10 * self + other }
}

theorem reduce_idempotent(r: Rat) {
    reduce(r.num, r.denom) = r
}

theorem neg_is_reduced(r: Rat) {
    is_reduced(-r.num, r.denom)
}

theorem neg_num(r: Rat) {
    r.neg.num = -r.num
}

theorem neg_denom(r: Rat) {
    r.neg.denom = r.denom
}

theorem add_zero_right(a: Rat) {
    a + Rat.0 = a
}

theorem add_zero_left(a: Rat) {
    Rat.0 + a = a
}

theorem add_comm(a: Rat, b: Rat) {
    a + b = b + a
}

theorem from_int_cancel(a: Int, b: Int) {
    Rat.from_int(a) = Rat.from_int(b) implies a = b
}

theorem mul_comm(a: Rat, b: Rat) {
    a * b = b * a
}

theorem rat_neg_one {
    -Rat.1 = Rat.from_int(-Int.1)
}

theorem mul_neg_one_right(r: Rat) {
    r * -Rat.1 = -r
}

theorem mul_neg_one_left(r: Rat) {
    -Rat.1 * r = -r
}

theorem mul_one_right(r: Rat) {
    r * Rat.1 = r
}

theorem mul_one_left(r: Rat) {
    Rat.1 * r = r
}

theorem mul_int_eq_int_mul(a: Int, b: Int) {
    Rat.from_int(a) * Rat.from_int(b) = Rat.from_int(a * b)
}

theorem unreduce_right(a: Int, b: Int) {
    b != Int.0 implies exists(d: Int) {
        reduce(a, b).num * d = a and reduce(a, b).denom * d = b
    }
}

theorem unreduce_left(a: Int, b: Int) {
    b != Int.0 implies exists(d: Int) {
        d * reduce(a, b).num = a and d * reduce(a, b).denom = b
    }
}

theorem mul_int_right(r: Rat, n: Int) {
    r * Rat.from_int(n) = reduce(r.num * n, r.denom)
}

theorem mul_int_left(n: Int, r: Rat) {
    Rat.from_int(n) * r = reduce(n * r.num, r.denom)
}

theorem cross_eq_imp_reduce_eq(a: Int, b: Int, c: Int, d: Int) {
    cross_equals(a, b, c, d) implies reduce(a, b) = reduce(c, d)
}

theorem reduce_eq_imp_cross_eq(a: Int, b: Int, c: Int, d: Int) {
    b != Int.0 and d != Int.0 and reduce(a, b) = reduce(c, d)
        implies cross_equals(a, b, c, d)
}

theorem mul_reduced_int_right(a: Int, b: Int, c: Int) {
    reduce(a, b) * Rat.from_int(c) = reduce(a * c, b)
}

theorem mul_reduced_int_left(a: Int, b: Int, c: Int) {
    Rat.from_int(c) * reduce(a, b) = reduce(c * a, b)
}

theorem mul_int_right_cancel(r1: Rat, r2: Rat, n: Int) {
    n != Int.0 and r1 * Rat.from_int(n) = r2 * Rat.from_int(n) implies r1 = r2
}

theorem mul_int_left_cancel(r1: Rat, r2: Rat, n: Int) {
    n != Int.0 and Rat.from_int(n) * r1 = Rat.from_int(n) * r2 implies r1 = r2
}

theorem mul_reduced_nondegen(a: Int, b: Int, c: Int, d: Int) {
    b != Int.0 and d != Int.0 implies
        reduce(a, b) * reduce(c, d) = reduce(a * c, b * d)
}

theorem reduce_zero_num(a: Int) {
    reduce(Int.0, a) = Rat.0
}

theorem mul_zero_right(r: Rat) {
    r * Rat.0 = Rat.0
}

theorem mul_reduced_degen(a: Int, b: Int, c: Int) {
    reduce(a, b) * reduce(c, Int.0) = reduce(a * c, b * Int.0)
}

theorem mul_reduced(a: Int, b: Int, c: Int, d: Int) {
    reduce(a, b) * reduce(c, d) = reduce(a * c, b * d)
}

theorem mul_assoc(a: Rat, b: Rat, c: Rat) {
    (a * b) * c = a * (b * c)
}

theorem add_int_eq_int_add(a: Int, b: Int) {
    Rat.from_int(a) + Rat.from_int(b) = Rat.from_int(a + b)
}

theorem reduce_cancels_right(a: Int, b: Int, c: Int) {
    c != Int.0 implies reduce(a, b) = reduce(a * c, b * c)
}

theorem reduce_cancels_left(a: Int, b: Int, c: Int) {
    c != Int.0 implies reduce(a, b) = reduce(c * a, c * b)
}

theorem add_reduce_right(r: Rat, a: Int, b: Int) {
    b != Int.0 implies r + reduce(a, b) = reduce(r.num * b + a * r.denom, r.denom * b)
}

theorem add_reduced(a: Int, b: Int, c: Int, d: Int) {
    b != Int.0 and d != Int.0 implies reduce(a, b) + reduce(c, d) = reduce(a * d + b * c, b * d)
}

theorem add_reduced_same_denom(a: Int, b: Int, c: Int) {
    reduce(a, c) + reduce(b, c) = reduce(a + b, c)
}

theorem add_assoc(a: Rat, b: Rat, c: Rat) {
    a + b + c = a + (b + c)
}

theorem common_denom(r1: Rat, r2: Rat) {
    exists(n1: Int, n2: Int, d: Int) {
        r1 = reduce(n1, d) and r2 = reduce(n2, d)
    }
}

theorem distrib_left(r1: Rat, r2: Rat, r3: Rat) {
    r1 * (r2 + r3) = r1 * r2 + r1 * r3
}

theorem distrib_right(r1: Rat, r2: Rat, r3: Rat) {
    (r1 + r2) * r3 = r1 * r3 + r2 * r3
}

theorem add_inv_cancels_right(a: Rat, b: Rat) {
    a + -a = Rat.0
}

theorem add_inv_cancels_left(a: Rat, b: Rat) {
    -a + a = Rat.0
}

// The additive algebraic structure.


instance Rat: AddSemigroup


instance Rat: AddCommSemigroup


instance Rat: AddMonoid


instance Rat: AddCommMonoid


instance Rat: AddGroup


instance Rat: AddCommGroup

theorem sub_self(a: Rat) {
    a - a = Rat.0
}

theorem reduce_one_one {
    reduce(Int.1, Int.1) = Rat.1
}

theorem reduce_self(n: Int) {
    n != Int.0 implies reduce(n, n) = Rat.1
}

theorem mul_inv_cancels_right(a: Rat) {
    a != Rat.0 implies a * a.inverse = Rat.1
}

theorem mul_inv_cancels_left(a: Rat) {
    a != Rat.0 implies a.inverse * a = Rat.1
}

theorem sub_zero(a: Rat) {
    a - Rat.0 = a
}

theorem not_pos_and_neg(a: Rat) {
    a.is_positive implies not a.is_negative
}

theorem reduce_pos_pos(a: Int, b: Int) {
    a.is_positive and b.is_positive implies reduce(a, b).is_positive
}

theorem add_pos_pos(a: Rat, b: Rat) {
    a.is_positive and b.is_positive implies (a + b).is_positive
}

theorem mul_pos_pos(a: Rat, b: Rat) {
    a.is_positive and b.is_positive implies (a * b).is_positive
}

theorem pos_inverse(a: Rat) {
    a.is_positive implies a.inverse.is_positive
}

theorem add_cancels_sub(a: Rat, b: Rat) {
    a - b + b = a
}

theorem neg_neg_is_pos(a: Rat) {
    a.is_negative implies (-a).is_positive
}

theorem neg_pos_is_neg(a: Rat) {
    a.is_positive implies (-a).is_negative
}

theorem zero_minus(a: Rat) {
    Rat.0 - a = -a
}

theorem mul_cancels_div(a: Rat, b: Rat) {
    b != Rat.0 implies (a / b) * b = a
}

/// True if this rational is less than or equal to the other.
instance Rat: LTE {
    define lte(self, other: Rat) -> Bool {
        (other - self).is_positive or self = other
    }
}


theorem rat_is_reflexive {
    is_reflexive(Rat.lte)
}

theorem lte_trans(a: Rat, b: Rat, c: Rat) {
    a <= b and b <= c implies a <= c
}

theorem rat_is_transitive {
    is_transitive(Rat.lte)
}

theorem lte_antisymm(a: Rat, b: Rat) {
    a <= b and b <= a implies a = b
}

theorem rat_is_antisymmetric {
    is_antisymmetric(Rat.lte)
}


instance Rat: PartialOrder

theorem pos_imp_zero_lt(a: Rat) {
    a.is_positive implies Rat.0 < a
}

theorem zero_lt_imp_pos(a: Rat) {
    Rat.0 < a implies a.is_positive
}

theorem lt_trans(a: Rat, b: Rat, c: Rat) {
    a < b and b < c implies a < c
}

theorem neg_imp_lt_zero(a: Rat) {
    a.is_negative implies a < Rat.0
}

theorem lt_zero_imp_neg(a: Rat) {
    a < Rat.0 implies a.is_negative
}

theorem two_neq_zero {
    Rat.2 != Rat.0
}

theorem times_two(r: Rat) {
    Rat.2 * r = r + r
}

theorem mul_cancels_right(a: Rat, b: Rat, c: Rat) {
    c != Rat.0 and a * c = b * c implies a = b
}

theorem half_plus_half(r: Rat) {
    (r / Rat.2) + (r / Rat.2) = r
}

theorem neg_mul(a: Rat, b: Rat) {
    (-a) * b = -(a * b)
}

theorem mul_neg_pos(a: Rat, b: Rat) {
    a.is_negative and b.is_positive implies (a * b).is_negative
}

theorem mul_pos_neg(a: Rat, b: Rat) {
    a.is_positive and b.is_negative implies (a * b).is_negative
}

theorem mul_neg_neg(a: Rat, b: Rat) {
    a.is_negative and b.is_negative implies (a * b).is_positive
}

theorem one_is_pos {
    Rat.1.is_positive
}

theorem two_is_pos {
    Rat.2.is_positive
}

theorem half_is_pos {
    Rat.2.inverse.is_positive
}

theorem sub_add_quasi_cancel(a: Rat, b: Rat) {
    a - (a + b) = -b
}

theorem not_lt_self(a: Rat) {
    not a < a
}

theorem not_lt_both_ways(a: Rat, b: Rat) {
    a < b implies not b < a
}

theorem lt_add_pos(a: Rat, b: Rat) {
    b.is_positive implies a < a + b
}

attributes Rat {
    /// The absolute value of a rational number.
    define abs(self) -> Rat {
        if self.is_negative {
            -self
        } else {
            self
        }
    }
}

theorem neg_sub(a: Rat, b: Rat) {
    -(a - b) = b - a
}

theorem single_trichotomy(a: Rat) {
    a.is_positive or a.is_negative or a = Rat.0
}

theorem trichotomy(a: Rat, b: Rat) {
    a < b or a = b or a > b
}

theorem lt_add_right(a: Rat, b: Rat, c: Rat) {
    a < b implies a + c < b + c
}

theorem add_neg_lt(a: Rat, b: Rat) {
    a.is_negative implies b + a < b
}

theorem adding_lts(a: Rat, b: Rat, c: Rat, d: Rat) {
    a < b and c < d implies a + c < b + d
}

theorem minus_cancels_plus(a: Rat, b: Rat) {
    a + b - b = a
}

theorem gt_minus_pos(a: Rat, b: Rat) {
    b.is_positive implies a > a - b
}

theorem sub_add(a: Rat, b: Rat, c: Rat) {
    a - (b + c) = a - b - c
}

theorem no_greatest(a: Rat) {
    exists(b: Rat) {
        a < b
    }
}

theorem sub_from_int(a: Int, b: Int) {
    Rat.from_int(a) - Rat.from_int(b) = Rat.from_int(a - b)
}

theorem lt_from_int(a: Int, b: Int) {
    a < b implies Rat.from_int(a) < Rat.from_int(b)
}

theorem lte_from_int(a: Int, b: Int) {
    a <= b implies Rat.from_int(a) <= Rat.from_int(b)
}

theorem from_int_lte_cancel(a: Int, b: Int) {
    Rat.from_int(a) <= Rat.from_int(b) implies a <= b
}

theorem from_int_lt_cancel(a: Int, b: Int) {
    Rat.from_int(a) < Rat.from_int(b) implies a < b
}

theorem gt_from_int(a: Int, b: Int) {
    a > b implies Rat.from_int(a) > Rat.from_int(b)
}

theorem gte_from_int(a: Int, b: Int) {
    a >= b implies Rat.from_int(a) >= Rat.from_int(b)
}

theorem lt_mul_pos(a: Rat, b: Rat, c: Rat) {
    a < b and c.is_positive implies a * c < b * c
}

theorem lte_mul_pos(a: Rat, b: Rat, c: Rat) {
    a <= b and c.is_positive implies a * c <= b * c
}

theorem lt_div_pos(a: Rat, b: Rat, c: Rat) {
    c.is_positive and a < b implies a / c < b / c
}

theorem lte_div_pos(a: Rat, b: Rat, c: Rat) {
    c.is_positive and a <= b implies a / c <= b / c
}

theorem mul_div_cancels(a: Rat, b: Rat) {
    b != Rat.0 implies (a * b) / b = a
}

theorem div_from_int(a: Rat) {
    Rat.from_int(a.num) / Rat.from_int(a.denom) = a
}

theorem lt_from_int_mul_denom(a: Rat, n: Int) {
    a.num < n * a.denom implies a < Rat.from_int(n)
}

theorem lte_from_int_mul_denom(a: Rat, n: Int) {
    a.num <= n * a.denom implies a <= Rat.from_int(n)
}

theorem floor_exists(a: Rat) {
    exists(q: Int) {
        Rat.from_int(q) <= a and a < Rat.from_int(q + Int.1)
    }
}

let floor_impl(a: Rat) -> q: Int satisfy {
    Rat.from_int(q) <= a and a < Rat.from_int(q + Int.1)
}

attributes Rat {
    /// The floor of a rational number (the greatest integer less than or equal to it).
    define floor(self) -> Int {
        floor_impl(self)
    }
}

theorem mul_lt_lt(a: Rat, b: Rat, c: Rat, d: Rat) {
    a.is_positive and c.is_positive and a < b and c < d implies a * c < b * d
}

theorem mul_lt_lte(a: Rat, b: Rat, c: Rat, d: Rat) {
    a.is_positive and c.is_positive and a < b and c <= d implies a * c < b * d
}

theorem pos_lte(a: Rat, b: Rat) {
    a.is_positive and a <= b implies b.is_positive
}

theorem mul_lte_lt(a: Rat, b: Rat, c: Rat, d: Rat) {
    a.is_positive and c.is_positive and a <= b and c < d implies a * c < b * d
}

theorem lt_pos_inverse(a: Rat, b: Rat) {
    a.is_positive and a < b implies b.inverse < a.inverse
}

theorem inverse_inverts(a: Rat) {
    a != Rat.0 implies a.inverse.inverse = a
}

theorem smaller_int_inverse(a: Rat) {
    a.is_positive implies exists(n: Int) {
        n.is_positive and Rat.from_int(n).inverse < a
    }
}

theorem gte_some_int(a: Rat) {
    exists(n: Int) {
        a >= Rat.from_int(n)
    }
}

theorem mul_denom(r: Rat) {
    r * Rat.from_int(r.denom) = Rat.from_int(r.num)
}

theorem half_pos(r: Rat) {
    r.is_positive implies (r / Rat.2).is_positive
}

theorem add_half_half(r: Rat) {
    (r / Rat.2) + (r / Rat.2) = r
}

theorem lt_neg(p: Rat, q: Rat) {
    p < q implies -q < -p
}

theorem lt_lte_trans(a: Rat, b: Rat, c: Rat) {
    a < b and b <= c implies a < c
}

theorem lte_lt_trans(a: Rat, b: Rat, c: Rat) {
    a <= b and b < c implies a < c
}

theorem lt_imp_rat_between(a: Rat, b: Rat) {
    a < b implies exists(c: Rat) {
        a < c and c < b
    }
}

theorem gt_imp_rat_between(a: Rat, b: Rat) {
    a > b implies exists(c: Rat) {
        a > c and c > b
    }
}

theorem lt_cancel_pos_mul_right(a: Rat, b: Rat, c: Rat) {
    c.is_positive and a * c < b * c implies a < b
}

theorem lt_cancel_pos_mul_left(a: Rat, b: Rat, c: Rat) {
    c.is_positive and c * a < c * b implies a < b
}

theorem gt_cancel_pos_mul_right(a: Rat, b: Rat, c: Rat) {
    c.is_positive and a * c > b * c implies a > b
}

theorem gt_cancel_pos_mul_left(a: Rat, b: Rat, c: Rat) {
    c.is_positive and c * a > c * b implies a > b
}

theorem cancel_positivity_left(a: Rat, b: Rat) {
    a.is_positive and (a * b).is_positive implies b.is_positive
}

theorem cancel_positivity_right(a: Rat, b: Rat) {
    b.is_positive and (a * b).is_positive implies a.is_positive
}

theorem mul_cancels_div_left(a: Rat, b: Rat) {
    b != Rat.0 implies b * (a / b) = a
}

theorem neg_abs(a: Rat) {
    (-a).abs = a.abs
}

theorem abs_non_pos(a: Rat) {
    not a.is_positive implies a.abs = -a
}

theorem abs_non_neg(a: Rat) {
    not a.is_negative implies a.abs = a
}

theorem abs_mul_abs(a: Rat, b: Rat) {
    (a * b).abs = (a * b.abs).abs
}

theorem mul_non_neg(a: Rat, b: Rat) {
    not a.is_negative and not b.is_negative implies not (a * b).is_negative
}

theorem mul_two_abs(a: Rat, b: Rat) {
    a.abs * b.abs = (a * b).abs
}

theorem abs_zero_imp_zero(a: Rat) {
    a.abs = Rat.0 implies a = Rat.0
}

theorem zero_lte_abs(a: Rat) {
    Rat.0 <= a.abs
}

theorem pos_inverses_lt(p: Rat, q: Rat, r: Rat) {
    p.is_positive and q.is_positive and r.is_positive
    and r / p < r / q
    implies
    q < p
}

theorem inverses_eq(p: Rat, q: Rat, r: Rat) {
    p != Rat.0 and q != Rat.0 and r != Rat.0
    and r / p = r / q
    implies
    q = p
}

theorem pos_inverses_lte(p: Rat, q: Rat, r: Rat) {
    p.is_positive and q.is_positive and r.is_positive
    and r / p <= r / q
    implies
    q <= p
}

theorem pos_inverses_gt(p: Rat, q: Rat, r: Rat) {
    p.is_positive and q.is_positive and r.is_positive
    and r / p > r / q
    implies
    q > p
}

theorem pos_inverses_gte(p: Rat, q: Rat, r: Rat) {
    p.is_positive and q.is_positive and r.is_positive
    and r / p >= r / q
    implies
    q >= p
}

theorem lt_cancel_add_right(p: Rat, q: Rat, r: Rat) {
    p + r < q + r implies p < q
}

theorem sub_distrib(p: Rat, q: Rat, r: Rat) {
    p * (q - r) = p * q - p * r
}

theorem half_lt_one {
    Rat.2.inverse < Rat.1
}

theorem lower_squared(a: Rat) {
    a.is_positive
    implies
    exists(eps: Rat) {
        eps.is_positive and eps * eps < a
    }
}

theorem square_lt_imp_lt(a: Rat, b: Rat) {
    a.is_positive and b.is_positive and a * a < b * b
    implies
    a < b
}

theorem smaller_positive(a: Rat) {
    a.is_positive implies
    exists(r: Rat) {
        r.is_positive and r < a
    }
}

theorem lt_both_pos(a: Rat, b: Rat) {
    a.is_positive and b.is_positive implies
    exists(r: Rat) {
        r.is_positive and r < a and r < b
    }
}

theorem lt_rhs_div_pos(a: Rat, b: Rat, c: Rat) {
    c.is_positive and a < b / c implies a * c < b
}

attributes Rat {
    /// True if the absolute difference between two rationals is less than epsilon.
    define is_close(self, other: Rat, eps: Rat) -> Bool {
        (self - other).abs < eps
    }
}

theorem lte_abs(q: Rat) {
    q <= q.abs
}

theorem close_comm(a: Rat, b: Rat, eps: Rat) {
    a.is_close(b, eps) implies b.is_close(a, eps)
}

theorem close_imp_bounds(a: Rat, b: Rat, eps: Rat) {
    a.is_close(b, eps) implies a < b + eps and a > b - eps
}

theorem bounds_imp_close(a: Rat, b: Rat, eps: Rat) {
    a < b + eps and a > b - eps implies a.is_close(b, eps)
}

attributes Rat {
    /// Converts a natural number to a rational number.
    let from_nat: Nat -> Rat = function(n: Nat) {
        Rat.from_int(Int.from_nat(n))
    }
}

theorem nat_lt_imp_rat_lt(a: Nat, b: Nat) {
    a < b implies Rat.from_nat(a) < Rat.from_nat(b)
}

theorem from_nat_nonneg(n: Nat) {
    Rat.0 <= Rat.from_nat(n)
}

theorem from_nat_add(a: Nat, b: Nat) {
    Rat.from_nat(a) + Rat.from_nat(b) = Rat.from_nat(a + b)
}

theorem from_nat_mul(a: Nat, b: Nat) {
    Rat.from_nat(a) * Rat.from_nat(b) = Rat.from_nat(a * b)
}

theorem nat_lte_imp_rat_lte(a: Nat, b: Nat) {
    a <= b implies Rat.from_nat(a) <= Rat.from_nat(b)
}

theorem recip_eq_one_div(a: Rat) {
    a != Rat.0 implies a.inverse = Rat.1 / a
}
 

// rat_props.ac
// It will be convenient to have a particular function that
// approaches zero.
define iop(n: Nat) -> Rat {
    Rat.1 / (Rat.1 + Rat.from_nat(n))
}

theorem iop_pos(n: Nat) {
    iop(n).is_positive
}

theorem pos_ne_zero(a: Rat) {
    a.is_positive implies a != Rat.0
}

theorem iop_ne_zero(n: Nat) {
    iop(n) != Rat.0
}

theorem iop_recip(n: Nat) {
    iop(n).inverse = Rat.1 + Rat.from_nat(n)
}

theorem iop_mul_lt_one(n: Nat) {
    iop(n) * Rat.from_nat(n) < Rat.1
}

theorem pos_lte_num(a: Rat) {
    a.is_positive implies a <= Rat.from_int(a.num)
}

theorem lt_some_int(a: Rat) {
    exists(n: Int) {
        a < Rat.from_int(n)
    }
}

theorem lt_some_nat(a: Rat) {
    exists(n: Nat) {
        a < Rat.from_nat(n)
    }
}

theorem iop_gets_lt(eps: Rat) {
    eps.is_positive implies exists(n: Nat) {
        forall(i: Nat) {
            n <= i implies iop(i) < eps
        }
    }
}

theorem three_is_positive {
    Rat.3.is_positive
}

theorem times_three(x: Rat) {
    Rat.3 * x = x + x + x
}

theorem three_thirds(x: Rat) {
    (x / Rat.3) + (x / Rat.3) + (x / Rat.3) = x
}

theorem some_mul_lt(a: Rat, b: Rat) {
    Rat.0 <= a and b.is_positive implies
    exists(eps: Rat) {
        eps.is_positive and a * eps < b
    }
}

theorem close_mul_pos(a: Rat, b: Rat, eps: Rat, r: Rat) {
    r.is_positive and a.is_close(b, eps)
    implies
    (a * r).is_close(b * r, eps * r)
}

theorem close_neg(a: Rat, b: Rat, eps: Rat) {
    a.is_close(b, eps) implies (-a).is_close(-b, eps)
}

theorem lte_mul_nonneg(a: Rat, b: Rat, c: Rat) {
    a <= b and Rat.0 <= c implies a * c <= b * c
}

theorem bounding_both(a: Rat, b: Rat) {
    exists(c: Rat) {
        a < c and b < c
    }
}

theorem bound_yields_finite_seq_abs_bounded(a: Nat -> Rat, n: Nat, bound: Rat) {
    forall(i: Nat) {
        i <= n implies a(i).abs < bound
    }
    implies
    exists(witness: Rat) {
        forall(i: Nat) {
            i <= n implies a(i).abs < witness
        }
    }
}

theorem finite_seq_abs_bounded(a: Nat -> Rat, n: Nat) {
    exists(bound: Rat) {
        forall(i: Nat) {
            i <= n implies a(i).abs < bound
        }
    }
}

theorem abs_reduce_left(a: Int, b: Int) {
    reduce(a.abs, b).abs = reduce(a, b).abs
}

theorem reduce_neg_num(a: Int, b: Int) {
    reduce(-a, b) = -reduce(a, b)
}

theorem reduce_neg_denom(a: Int, b: Int) {
    reduce(a, -b) = -reduce(a, b)
}

theorem abs_reduce_right(a: Int, b: Int) {
    reduce(a, b.abs).abs = reduce(a, b).abs
}

theorem reduce_nonneg(a: Int, b: Int) {
    not a.is_negative and not b.is_negative
    implies
    not reduce(a, b).is_negative
}

/// A positive reduction has numerator bounded by the unreduced numerator.
theorem reduce_num_lte_raw_of_positive(a: Int, b: Int) {
    reduce(a, b).is_positive and b.is_positive implies reduce(a, b).num <= a
}

theorem reduce_abs(a: Int, b: Int) {
    reduce(a.abs, b.abs) = reduce(a, b).abs
}

theorem lte_cancel_mul_pos(p: Rat, q: Rat, r: Rat) {
    r.is_positive and p * r <= q * r
    implies
    p <= q
}

theorem reduce_lte(a: Int, b: Int, c: Int) {
    not c.is_negative and a <= b implies
    reduce(a, c) <= reduce(b, c)
}

theorem triangle_ineq(a: Rat, b: Rat) {
    (a + b).abs <= a.abs + b.abs
}

// Not all that generally useful.
theorem diff_mul_bound(a0: Rat, b0: Rat, a1: Rat, b1: Rat) {
    (a0 * b0 - a1 * b1).abs <= a0.abs * (b0 - b1).abs + b1.abs * (a0 - a1).abs
}

theorem nonneg_lt_imp_pos(a: Rat, b: Rat) {
    not a.is_negative and a < b implies
    b.is_positive
}

theorem lt_pos_mul_lt_pos(a: Rat, b: Rat, c: Rat, d: Rat) {
    not a.is_negative and not c.is_negative and a < b and c < d
    implies
    a * c < b * d
}

theorem abs_nonneg(a: Rat) {
    not a.abs.is_negative
}

theorem lte_mul_lte(a: Rat, b: Rat, c: Rat, d: Rat) {
    not a.is_negative and not c.is_negative and a <= b and c <= d
    implies
    a * c <= b * d
}

theorem add_div_distrib(a: Rat, b: Rat, c: Rat) {
    (a + b) / c = a / c + b / c
}

theorem sub_div_distrib(a: Rat, b: Rat, c: Rat) {
    (a - b) / c = a / c - b / c
}

theorem recip_mul(a: Rat, b: Rat) {
    (a * b).inverse = a.inverse * b.inverse
}

theorem mul_fractions(a: Rat, b: Rat, c: Rat, d: Rat) {
    (a / b) * (c / d) = (a * c) / (b * d)
}

theorem cancel_left_num_denom(a: Rat, b: Rat, c: Rat) {
    a != Rat.0 implies
    (a * b) / (a * c) = b / c
}

theorem cancel_to_inverse(a: Rat, b: Rat) {
    a != Rat.0 and b != Rat.0 implies
    a / (a * b) = b.inverse
}

theorem recip_diff(a: Rat, b: Rat) {
    a != Rat.0 and b != Rat.0 implies
    a.inverse - b.inverse = (b - a) / (a * b)
}
theorem lt_mul_lte(a: Rat, b: Rat, c: Rat, d: Rat) {
    not a.is_negative and c.is_positive and a < b and c <= d
    implies
    a * c < b * d
}

// This isn't ideal but this is how we have defined division.
theorem div_zero(a: Rat) {
    a / Rat.0 = Rat.0
}

theorem recip_recip(a: Rat) {
    a.inverse.inverse = a
}

theorem neg_recip(a: Rat) {
    a.is_negative implies a.inverse.is_negative
}

theorem div_neg_neg(a: Rat, b: Rat) {
    a.is_negative and b.is_negative
    implies
    (a / b).is_positive
}

theorem div_negs_cancel(a: Rat, b: Rat) {
    a / b = (-a) / (-b)
}

theorem abs_div(a: Rat, b: Rat) {
    b != Rat.0 implies
    (a / b).abs = a.abs / b.abs
}

theorem lt_make_left_denom(a: Rat, b: Rat, c: Rat) {
    c.is_positive and a < b * c implies a / c < b
}

theorem lt_make_right_denom(a: Rat, b: Rat, c: Rat) {
    b.is_positive and a * b < c implies a < c / b
}

theorem lt_elim_left_denom(a: Rat, b: Rat, c: Rat) {
    b.is_positive and a / b < c implies a < c * b
}

theorem cross_mul_lt(a: Rat, b: Rat, c: Rat, d: Rat) {
    b.is_positive and d.is_positive and a * d < c * b
    implies
    a / b < c / d
}

theorem cross_mul_lte(a: Rat, b: Rat, c: Rat, d: Rat) {
    b.is_positive and d.is_positive and a * d <= c * b
    implies
    a / b <= c / d
}

theorem abs_of_div(a: Rat, b: Rat) {
    (a / b).abs = a.abs / b.abs
}

theorem mul_div_swap(a: Rat, b: Rat, c: Rat) {
    a * b / c = (a / c) * b
}

theorem zero_recip(a: Rat) {
    Rat.0.inverse = Rat.0
}

// The multiplicative algebraic structure.


instance Rat: Semigroup


instance Rat: Monoid


instance Rat: Semiring


instance Rat: Ring


instance Rat: CommSemigroup


instance Rat: CommMonoid


instance Rat: CommRing


theorem zero_is_different_than_one {
    Rat.0 != Rat.1
}

instance Rat: Field

theorem rat_total(a: Rat, b: Rat) { a <= b or b <= a }

theorem lte_add_right(a: Rat, b: Rat, c: Rat) {
    a <= b implies a + c <= b + c
}

theorem lte_add_left(a: Rat, b: Rat, c: Rat) {
    b <= c implies a + b <= a + c
}

instance Rat: LinearOrder


instance Rat: Meet {
    define meet(self, other: Rat) -> Rat {
        self.min(other)
    }
}

instance Rat: Join {
    define join(self, other: Rat) -> Rat {
        self.max(other)
    }
}

theorem rat_meet_lte_left(a: Rat, b: Rat) {
    a.meet(b) <= a
}

theorem rat_meet_lte_right(a: Rat, b: Rat) {
    a.meet(b) <= b
}

theorem rat_lte_meet_of_bounds(c: Rat, a: Rat, b: Rat) {
    c <= a and c <= b implies c <= a.meet(b)
}

instance Rat: MeetSemilattice

theorem rat_lte_join_left(a: Rat, b: Rat) {
    a <= a.join(b)
}

theorem rat_lte_join_right(a: Rat, b: Rat) {
    b <= a.join(b)
}

theorem rat_join_lte_of_bounds(a: Rat, b: Rat, c: Rat) {
    a <= c and b <= c implies a.join(b) <= c
}

instance Rat: JoinSemilattice

instance Rat: Lattice

theorem rat_meet_join_distrib_left(a: Rat, b: Rat, c: Rat) {
    a.meet(b.join(c)) = a.meet(b).join(a.meet(c))
}

theorem rat_join_meet_distrib_left(a: Rat, b: Rat, c: Rat) {
    a.join(b.meet(c)) = a.join(b).meet(a.join(c))
}

instance Rat: DistribLattice


instance Rat: AddLeftOrderedGroup
instance Rat: AddOrderedGroup


theorem mul_nonnegative(a: Rat, b: Rat) {
    Rat.0 <= a and Rat.0 <= b implies Rat.0 <= a * b
}

instance Rat: OrderedField

// rat_countable.ac
/// The rational represented by an integer numerator and a positive natural
/// denominator encoded as its predecessor.
define rat_pair_enum(pair: Pair[Int, Nat]) -> Rat {
    reduce(pair.first, Int.from_nat(pair.second.suc))
}

/// The standard zig-zag enumeration of the integers:
/// `0, -1, 1, -2, 2, ...`.
define int_zigzag(n: Nat) -> Int {
    match n {
        Nat.zero {
            Int.0
        }
        Nat.suc(pred) {
            match int_zigzag(pred) {
                Int.from_nat(k) {
                    Int.neg_suc(k)
                }
                Int.neg_suc(k) {
                    Int.from_nat(k.suc)
                }
            }
        }
    }
}

theorem int_zigzag_suc_of_from_nat(n: Nat, k: Nat) {
    int_zigzag(n) = Int.from_nat(k) implies int_zigzag(n.suc) = Int.neg_suc(k)
}

theorem int_zigzag_suc_of_neg_suc(n: Nat, k: Nat) {
    int_zigzag(n) = Int.neg_suc(k) implies int_zigzag(n.suc) = Int.from_nat(k.suc)
}

/// Inductive predicate that the zig-zag enumeration hits both `k` and `-(k+1)`.
define int_zigzag_hits_pair(k: Nat) -> Bool {
    exists(nonneg_idx: Nat) {
        int_zigzag(nonneg_idx) = Int.from_nat(k)
    } and exists(neg_idx: Nat) {
        int_zigzag(neg_idx) = Int.neg_suc(k)
    }
}

theorem int_zigzag_hits_pair_zero {
    int_zigzag_hits_pair(Nat.0)
}

theorem int_zigzag_hits_pair_step(k: Nat) {
    int_zigzag_hits_pair(k) implies int_zigzag_hits_pair(k.suc)
}

theorem int_zigzag_hits_pair_all(k: Nat) {
    int_zigzag_hits_pair(k)
}

theorem int_zigzag_surjective {
    is_surjective_fn(int_zigzag)
}

/// The next pair in the diagonal enumeration of pairs of natural numbers.
define nat_pair_next(pair: Pair[Nat, Nat]) -> Pair[Nat, Nat] {
    match pair.second {
        Nat.zero {
            Pair.new(Nat.0, pair.first.suc)
        }
        Nat.suc(pred) {
            Pair.new(pair.first.suc, pred)
        }
    }
}

/// The diagonal enumeration of pairs of natural numbers:
/// `(0,0), (0,1), (1,0), (0,2), ...`.
define nat_pair_zigzag(n: Nat) -> Pair[Nat, Nat] {
    match n {
        Nat.zero {
            Pair.new(Nat.0, Nat.0)
        }
        Nat.suc(pred) {
            nat_pair_next(nat_pair_zigzag(pred))
        }
    }
}

theorem nat_pair_next_zero_second(a: Nat) {
    nat_pair_next(Pair.new(a, Nat.0)) = Pair.new(Nat.0, a.suc)
}

theorem nat_pair_next_suc_second(a: Nat, b: Nat) {
    nat_pair_next(Pair.new(a, b.suc)) = Pair.new(a.suc, b)
}

/// Predicate that a pair of natural numbers occurs in the diagonal enumeration.
define nat_pair_zigzag_hits_pair(a: Nat, b: Nat) -> Bool {
    exists(n: Nat) {
        nat_pair_zigzag(n) = Pair.new(a, b)
    }
}

theorem nat_pair_zigzag_hits_pair_transport(a: Nat, b: Nat, c: Nat, d: Nat) {
    a = c and b = d and nat_pair_zigzag_hits_pair(a, b) implies
        nat_pair_zigzag_hits_pair(c, d)
}

theorem nat_pair_zigzag_hits_zero_zero {
    nat_pair_zigzag_hits_pair(Nat.0, Nat.0)
}

theorem nat_pair_zigzag_hit_step_down(a: Nat, b: Nat) {
    nat_pair_zigzag_hits_pair(a, b.suc) implies
        nat_pair_zigzag_hits_pair(a.suc, b)
}

theorem nat_pair_zigzag_hit_next_diagonal(d: Nat) {
    nat_pair_zigzag_hits_pair(d, Nat.0) implies
        nat_pair_zigzag_hits_pair(Nat.0, d.suc)
}

theorem sub_suc_on_bounded_diagonal(d: Nat, a: Nat) {
    a.suc <= d implies d - a = (d - a.suc).suc
}

theorem nat_pair_zigzag_diagonal_from_zero(d: Nat, a: Nat) {
    nat_pair_zigzag_hits_pair(Nat.0, d) and a <= d implies
        nat_pair_zigzag_hits_pair(a, d - a)
}

/// Predicate that the whole diagonal with coordinate sum `d` occurs.
define nat_pair_zigzag_hits_diagonal(d: Nat) -> Bool {
    forall(a: Nat) {
        a <= d implies nat_pair_zigzag_hits_pair(a, d - a)
    }
}

theorem nat_pair_zigzag_hits_diagonal_at(d: Nat, a: Nat) {
    nat_pair_zigzag_hits_diagonal(d) and a <= d implies
        nat_pair_zigzag_hits_pair(a, d - a)
}

theorem nat_pair_zigzag_hits_diagonal_zero {
    nat_pair_zigzag_hits_diagonal(Nat.0)
}

theorem nat_pair_zigzag_hits_diagonal_step(d: Nat) {
    nat_pair_zigzag_hits_diagonal(d) implies nat_pair_zigzag_hits_diagonal(d.suc)
}

theorem nat_pair_zigzag_hits_diagonal_all(d: Nat) {
    nat_pair_zigzag_hits_diagonal(d)
}

theorem nat_pair_zigzag_hits(a: Nat, b: Nat) {
    nat_pair_zigzag_hits_pair(a, b)
}

theorem nat_pair_zigzag_surjective {
    is_surjective_fn(nat_pair_zigzag)
}

/// Convert a natural pair into an integer-natural pair using the zig-zag
/// enumeration on the first coordinate.
define int_nat_pair_of_nat_pair(pair: Pair[Nat, Nat]) -> Pair[Int, Nat] {
    Pair.new(int_zigzag(pair.first), pair.second)
}

theorem int_nat_pair_of_nat_pair_new(n: Nat, d: Nat) {
    int_nat_pair_of_nat_pair(Pair.new(n, d)) = Pair.new(int_zigzag(n), d)
}

theorem int_nat_pair_of_nat_pair_surjective {
    is_surjective_fn(int_nat_pair_of_nat_pair)
}

/// The integer-natural pair enumeration obtained by composing the natural-pair
/// diagonal enumeration with the coordinate conversion.
define int_nat_pair_zigzag(n: Nat) -> Pair[Int, Nat] {
    int_nat_pair_of_nat_pair(nat_pair_zigzag(n))
}

theorem int_nat_pair_zigzag_surjective {
    is_surjective_fn(int_nat_pair_zigzag)
}

theorem from_nat_positive_eq_suc(k: Nat) {
    Int.from_nat(k).is_positive implies exists(n: Nat) {
        Int.from_nat(k) = Int.from_nat(n.suc)
    }
}

theorem neg_suc_not_positive(k: Nat) {
    not Int.neg_suc(k).is_positive
}

theorem positive_int_eq_from_nat_suc(a: Int) {
    a.is_positive implies exists(n: Nat) {
        a = Int.from_nat(n.suc)
    }
}

theorem rat_pair_enum_new(num: Int, denom_pred: Nat) {
    rat_pair_enum(Pair.new(num, denom_pred)) =
        reduce(num, Int.from_nat(denom_pred.suc))
}

theorem rat_pair_enum_surjective {
    is_surjective_fn(rat_pair_enum)
}

/// A natural-number enumeration of the rationals.
define rat_zigzag(n: Nat) -> Rat {
    rat_pair_enum(int_nat_pair_zigzag(n))
}

theorem rat_zigzag_surjective {
    is_surjective_fn(rat_zigzag)
}

theorem rational_numbers_are_denumerable {
    exists(enumeration: Nat -> Rat) {
        is_surjective_fn(enumeration)
    }
}

theorem int_from_nat_injective(a: Nat, b: Nat) {
    Int.from_nat(a) = Int.from_nat(b) implies a = b
}

theorem rat_from_nat_injective {
    is_injective_fn(Rat.from_nat)
}

theorem rat_zigzag_inverse_injective {
    is_injective_fn(inverse_fn(rat_zigzag))
}

theorem rational_numbers_have_countably_infinite_enumeration {
    exists(enumeration: Nat -> Rat) {
        is_surjective_fn(enumeration)
    } and exists(encoding: Rat -> Nat) {
        is_injective_fn(encoding)
    } and exists(embedding: Nat -> Rat) {
        is_injective_fn(embedding)
    }
}
