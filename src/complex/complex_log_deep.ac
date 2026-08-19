// ---------------------------------------------------------------------------
// Deepening of the complex logarithm and exponential.
//
// This module extends src/complex/complex_log.ac and src/complex/complex_exp.ac
// with the fundamental identities of the complex logarithm and exponential:
//
//   a. The special values (1).log = 0 and e.log = 1.
//   b. The inverse laws between the complex logarithm and the complex
//      exponential on the positive real axis and on the embedded real axis.
//      The real package now exports exp_pos, exp_log_or_zero, log_some_of_pos_exists,
//      log_one, log_mul and exp_injective through src/real/interface.ac, so the
//      inverse laws that were stated but commented out in complex_log.ac are
//      proved here in unconditional form.
//   c. The multiplicative law of the logarithm on positive reals.
//   d. The modulus and conjugation identities: multiplicativity of the modulus,
//      conjugation invariance of the modulus, the identity |z|^2 = z·conj(z),
//      and the distributivity of conjugation over products.
//   e. The field laws for the inverse and division: z·z^(-1) = 1, the inverse
//      of a product, z/z = 1, and the modulus of inverses and quotients.
//   f. The exponential identities: (-z).exp = z.exp^(-1), (z - w).exp = ...,
//      the lift of the real addition law, and the Euler identities.
// ---------------------------------------------------------------------------

from real import Real, pi, pos_imp_eq_abs, exp_pos, exp_log_or_zero, log_some_of_pos_exists, log_one, log_mul, exp_injective, mul_pos_pos, two
from algebra.field.field import inverse_dist
from complex.complex import Complex, conj_mul, conj_conj, conj_inverse, conj_div, real_add_lifts, mul_zero_left, mul_zero_right, add_neg_cancel, mul_left_cancel, zero_is_different_than_one, from_real_zero, from_real_one, from_real_is_real, re_from_real, im_from_real, div_self, abs_squared_conj
from complex.complex_abs import modulus_mul, modulus_conj, modulus_eq_zero, modulus_of_one, modulus_from_real, modulus_squared
from complex.complex_exp import complex_exp, complex_exp_from_real, complex_exp_add, complex_exp_zero, real_mul_left_cancel, real_one_positive, complex_i_mul_real, complex_exp_i_mul_real_eq, euler_identity
from complex.complex_log import complex_log, complex_log_from_real_pos
from complex.roots_of_unity import complex_exp_two_pi

// ---------------------------------------------------------------------------
// a. Special values of the complex logarithm.
// ---------------------------------------------------------------------------

/// The defaulting real logarithm recovers an exponential value:
/// (x.exp).log.get_or_else(Real.0) = x for every real x.
theorem log_value_exp(x: Real) {
    (x.exp).log.get_or_else(Real.0) = x
} by {
    exp_pos(x)
    x.exp > Real.0
    log_some_of_pos_exists(x.exp)
    let y: Real satisfy {
        x.exp.log = Option.some(y)
    }
    exp_log_or_zero(x.exp, y)
    x.exp > Real.0 and x.exp.log = Option.some(y) implies y.exp = x.exp
    y.exp = x.exp
    (x.exp).log.get_or_else(Real.0) = option_get_or_else(x.exp.log, Real.0)
    option_get_or_else_some[Real](y, Real.0)
    option_get_or_else(Option.some(y), Real.0) = y
    (x.exp).log.get_or_else(Real.0) = y
    ((x.exp).log.get_or_else(Real.0)).exp = y.exp
    ((x.exp).log.get_or_else(Real.0)).exp = x.exp
    exp_injective((x.exp).log.get_or_else(Real.0), x)
    ((x.exp).log.get_or_else(Real.0)).exp = x.exp implies (x.exp).log.get_or_else(Real.0) = x
    (x.exp).log.get_or_else(Real.0) = x
}

/// The complex logarithm of one is zero.
theorem complex_log_one {
    complex_log(Complex.1) = Complex.0
} by {
    complex_log(Complex.1) = Complex.from_real((Complex.1.modulus).log.get_or_else(Real.0))
    modulus_of_one
    Complex.1.modulus = Real.1
    (Complex.1.modulus).log.get_or_else(Real.0) = (Real.1).log.get_or_else(Real.0)
    log_one
    Real.1.log = Option.some(Real.0)
    (Real.1).log.get_or_else(Real.0) = option_get_or_else(Real.1.log, Real.0)
    option_get_or_else_some[Real](Real.0, Real.0)
    option_get_or_else(Option.some(Real.0), Real.0) = Real.0
    (Real.1).log.get_or_else(Real.0) = Real.0
    Complex.from_real((Complex.1.modulus).log.get_or_else(Real.0)) = Complex.from_real((Real.1).log.get_or_else(Real.0))
    Complex.from_real((Real.1).log.get_or_else(Real.0)) = Complex.from_real(Real.0)
    from_real_zero
    Complex.from_real(Real.0) = Complex.0
    Complex.from_real((Real.1).log.get_or_else(Real.0)) = Complex.0
    Complex.from_real((Complex.1.modulus).log.get_or_else(Real.0)) = Complex.0
    complex_log(Complex.1) = Complex.0
}

/// The complex logarithm of the real number e (the exponential of one) is one.
theorem complex_log_e {
    complex_log(Complex.from_real((Real.1).exp)) = Complex.1
} by {
    complex_log(Complex.from_real((Real.1).exp)) =
        Complex.from_real((Complex.from_real((Real.1).exp).modulus).log.get_or_else(Real.0))
    exp_pos(Real.1)
    (Real.1).exp > Real.0
    (Real.1).exp.is_positive
    pos_imp_eq_abs((Real.1).exp)
    (Real.1).exp = (Real.1).exp.abs
    modulus_from_real((Real.1).exp)
    Complex.from_real((Real.1).exp).modulus = (Real.1).exp.abs
    Complex.from_real((Real.1).exp).modulus = (Real.1).exp
    (Complex.from_real((Real.1).exp).modulus).log.get_or_else(Real.0) = ((Real.1).exp).log.get_or_else(Real.0)
    log_value_exp(Real.1)
    ((Real.1).exp).log.get_or_else(Real.0) = Real.1
    Complex.from_real((Complex.from_real((Real.1).exp).modulus).log.get_or_else(Real.0)) = Complex.from_real(Real.1)
    from_real_one
    Complex.from_real(Real.1) = Complex.1
    complex_log(Complex.from_real((Real.1).exp)) = Complex.1
}

// ---------------------------------------------------------------------------
// b. The inverse laws between the complex logarithm and the complex
//    exponential on the positive real axis and on the real axis.
// ---------------------------------------------------------------------------

/// The complex exponential of the complex logarithm of a positive real number
/// is the number itself: (x.log).exp = x for every positive real x.
theorem complex_exp_log_pos(x: Real) {
    x > Real.0 implies complex_exp(complex_log(Complex.from_real(x))) = Complex.from_real(x)
} by {
    if x > Real.0 {
        complex_log_from_real_pos(x)
        x > Real.0 implies complex_log(Complex.from_real(x)) = Complex.from_real(x.log.get_or_else(Real.0))
        complex_log(Complex.from_real(x)) = Complex.from_real(x.log.get_or_else(Real.0))
        complex_exp(complex_log(Complex.from_real(x))) = complex_exp(Complex.from_real(x.log.get_or_else(Real.0)))
        complex_exp_from_real(x.log.get_or_else(Real.0))
        complex_exp(Complex.from_real(x.log.get_or_else(Real.0))) = Complex.from_real((x.log.get_or_else(Real.0)).exp)
        log_some_of_pos_exists(x)
        let y: Real satisfy {
            x.log = Option.some(y)
        }
        exp_log_or_zero(x, y)
        x > Real.0 and x.log = Option.some(y) implies y.exp = x
        y.exp = x
        (x.log.get_or_else(Real.0)).exp = x
        Complex.from_real((x.log.get_or_else(Real.0)).exp) = Complex.from_real(x)
        complex_exp(complex_log(Complex.from_real(x))) = Complex.from_real(x)
    }
}

/// The complex logarithm of the complex exponential of an embedded real number
/// is the number itself: (x.exp).log = x for every real x.  This is the
/// restricted inverse property, holding on the whole real axis.
theorem complex_log_exp_real(x: Real) {
    complex_log(complex_exp(Complex.from_real(x))) = Complex.from_real(x)
} by {
    complex_exp_from_real(x)
    complex_exp(Complex.from_real(x)) = Complex.from_real(x.exp)
    complex_log(complex_exp(Complex.from_real(x))) = complex_log(Complex.from_real(x.exp))
    complex_log(Complex.from_real(x.exp)) =
        Complex.from_real((Complex.from_real(x.exp).modulus).log.get_or_else(Real.0))
    exp_pos(x)
    x.exp > Real.0
    x.exp.is_positive
    pos_imp_eq_abs(x.exp)
    x.exp = x.exp.abs
    modulus_from_real(x.exp)
    Complex.from_real(x.exp).modulus = x.exp.abs
    Complex.from_real(x.exp).modulus = x.exp
    (Complex.from_real(x.exp).modulus).log.get_or_else(Real.0) = (x.exp).log.get_or_else(Real.0)
    Complex.from_real((Complex.from_real(x.exp).modulus).log.get_or_else(Real.0)) =
        Complex.from_real((x.exp).log.get_or_else(Real.0))
    complex_log(Complex.from_real(x.exp)) = Complex.from_real((x.exp).log.get_or_else(Real.0))
    log_value_exp(x)
    (x.exp).log.get_or_else(Real.0) = x
    Complex.from_real((x.exp).log.get_or_else(Real.0)) = Complex.from_real(x)
    complex_log(complex_exp(Complex.from_real(x))) = Complex.from_real(x)
}

// ---------------------------------------------------------------------------
// c. The multiplicative law of the complex logarithm on positive reals.
// ---------------------------------------------------------------------------

/// The complex logarithm of a positive product is the sum of the logarithms:
/// (x·y).log = x.log + y.log for positive real x and y.
theorem complex_log_mul_pos(x: Real, y: Real) {
    x > Real.0 and y > Real.0 implies
        complex_log(Complex.from_real(x * y)) =
            complex_log(Complex.from_real(x)) + complex_log(Complex.from_real(y))
} by {
    if x > Real.0 and y > Real.0 {
        x.is_positive
        y.is_positive
        mul_pos_pos(x, y)
        x.is_positive and y.is_positive implies (x * y).is_positive
        (x * y).is_positive
        (x * y) > Real.0
        complex_log_from_real_pos(x * y)
        (x * y) > Real.0 implies complex_log(Complex.from_real(x * y)) = Complex.from_real((x * y).log.get_or_else(Real.0))
        complex_log(Complex.from_real(x * y)) = Complex.from_real((x * y).log.get_or_else(Real.0))
        complex_log_from_real_pos(x)
        x > Real.0 implies complex_log(Complex.from_real(x)) = Complex.from_real(x.log.get_or_else(Real.0))
        complex_log(Complex.from_real(x)) = Complex.from_real(x.log.get_or_else(Real.0))
        complex_log_from_real_pos(y)
        y > Real.0 implies complex_log(Complex.from_real(y)) = Complex.from_real(y.log.get_or_else(Real.0))
        complex_log(Complex.from_real(y)) = Complex.from_real(y.log.get_or_else(Real.0))
        complex_log(Complex.from_real(x)) + complex_log(Complex.from_real(y)) =
            Complex.from_real(x.log.get_or_else(Real.0)) + Complex.from_real(y.log.get_or_else(Real.0))
        log_some_of_pos_exists(x)
        let a: Real satisfy {
            x.log = Option.some(a)
        }
        log_some_of_pos_exists(y)
        let b: Real satisfy {
            y.log = Option.some(b)
        }
        log_mul(x, y, a, b)
        x > Real.0 and y > Real.0 and x.log = Option.some(a) and y.log = Option.some(b) implies (x * y).log = Option.some(a + b)
        (x * y).log = Option.some(a + b)
        x.log.get_or_else(Real.0) = option_get_or_else(x.log, Real.0)
        option_get_or_else_some[Real](a, Real.0)
        option_get_or_else(Option.some(a), Real.0) = a
        x.log.get_or_else(Real.0) = a
        y.log.get_or_else(Real.0) = option_get_or_else(y.log, Real.0)
        option_get_or_else_some[Real](b, Real.0)
        option_get_or_else(Option.some(b), Real.0) = b
        y.log.get_or_else(Real.0) = b
        (x * y).log.get_or_else(Real.0) = option_get_or_else((x * y).log, Real.0)
        option_get_or_else_some[Real](a + b, Real.0)
        option_get_or_else(Option.some(a + b), Real.0) = a + b
        (x * y).log.get_or_else(Real.0) = a + b
        (x * y).log.get_or_else(Real.0) = x.log.get_or_else(Real.0) + y.log.get_or_else(Real.0)
        Complex.from_real((x * y).log.get_or_else(Real.0)) = Complex.from_real(x.log.get_or_else(Real.0) + y.log.get_or_else(Real.0))
        real_add_lifts(x.log.get_or_else(Real.0), y.log.get_or_else(Real.0))
        Complex.from_real(x.log.get_or_else(Real.0) + y.log.get_or_else(Real.0)) =
            Complex.from_real(x.log.get_or_else(Real.0)) + Complex.from_real(y.log.get_or_else(Real.0))
        Complex.from_real((x * y).log.get_or_else(Real.0)) =
            Complex.from_real(x.log.get_or_else(Real.0)) + Complex.from_real(y.log.get_or_else(Real.0))
        complex_log(Complex.from_real(x * y)) =
            Complex.from_real(x.log.get_or_else(Real.0)) + Complex.from_real(y.log.get_or_else(Real.0))
        complex_log(Complex.from_real(x * y)) =
            complex_log(Complex.from_real(x)) + complex_log(Complex.from_real(y))
    }
}

// ---------------------------------------------------------------------------
// Structure of the complex logarithm: it is real-valued.
// ---------------------------------------------------------------------------

/// The real part of the complex logarithm is the real logarithm of the modulus.
theorem complex_log_re(z: Complex) {
    complex_log(z).re = (z.modulus).log.get_or_else(Real.0)
} by {
    complex_log(z) = Complex.from_real((z.modulus).log.get_or_else(Real.0))
    re_from_real((z.modulus).log.get_or_else(Real.0))
    Complex.from_real((z.modulus).log.get_or_else(Real.0)).re = (z.modulus).log.get_or_else(Real.0)
    complex_log(z).re = (z.modulus).log.get_or_else(Real.0)
}

/// The imaginary part of the complex logarithm is zero.
theorem complex_log_im(z: Complex) {
    complex_log(z).im = Real.0
} by {
    complex_log(z) = Complex.from_real((z.modulus).log.get_or_else(Real.0))
    im_from_real((z.modulus).log.get_or_else(Real.0))
    Complex.from_real((z.modulus).log.get_or_else(Real.0)).im = Real.0
    complex_log(z).im = Real.0
}

/// The complex logarithm of every complex number is real-valued.
theorem complex_log_is_real(z: Complex) {
    complex_log(z).is_real
} by {
    complex_log(z) = Complex.from_real((z.modulus).log.get_or_else(Real.0))
    from_real_is_real((z.modulus).log.get_or_else(Real.0))
    Complex.from_real((z.modulus).log.get_or_else(Real.0)).is_real
    complex_log(z).is_real
}

/// The modulus of the complex logarithm is the absolute value of the real
/// logarithm of the modulus.
theorem complex_log_modulus(z: Complex) {
    complex_log(z).modulus = (z.modulus).log.get_or_else(Real.0).abs
} by {
    complex_log(z) = Complex.from_real((z.modulus).log.get_or_else(Real.0))
    complex_log(z).modulus = Complex.from_real((z.modulus).log.get_or_else(Real.0)).modulus
    modulus_from_real((z.modulus).log.get_or_else(Real.0))
    Complex.from_real((z.modulus).log.get_or_else(Real.0)).modulus = (z.modulus).log.get_or_else(Real.0).abs
    complex_log(z).modulus = (z.modulus).log.get_or_else(Real.0).abs
}

// ---------------------------------------------------------------------------
// d. The modulus and conjugation identities.
// ---------------------------------------------------------------------------

/// The modulus is multiplicative: |z·w| = |z|·|w|.
theorem complex_modulus_product(a: Complex, b: Complex) {
    (a * b).modulus = a.modulus * b.modulus
} by {
    modulus_mul(a, b)
    (a * b).modulus = a.modulus * b.modulus
}

/// Conjugation preserves the modulus: |conj(z)| = |z|.
theorem complex_modulus_conj(z: Complex) {
    z.conj.modulus = z.modulus
} by {
    modulus_conj(z)
    z.conj.modulus = z.modulus
}

/// The conjugate of a product is the product of the conjugates:
/// conj(z·w) = conj(z)·conj(w).
theorem complex_conj_product(a: Complex, b: Complex) {
    (a * b).conj = a.conj * b.conj
} by {
    conj_mul(a, b)
    (a * b).conj = a.conj * b.conj
}

/// Complex conjugation is an involution: conj(conj(z)) = z.
theorem complex_conj_involution(z: Complex) {
    z.conj.conj = z
} by {
    conj_conj(z)
    z.conj.conj = z
}

/// The product of a complex number with its conjugate is the squared modulus:
/// z·conj(z) = |z|^2.
theorem complex_mul_conj_squared(z: Complex) {
    z * z.conj = Complex.from_real(z.modulus * z.modulus)
} by {
    abs_squared_conj(z)
    z * z.conj = Complex.new(z.abs_squared, Real.0)
    modulus_squared(z)
    z.modulus * z.modulus = z.abs_squared
    Complex.from_real(z.abs_squared) = Complex.new(z.abs_squared, Real.0)
    z * z.conj = Complex.from_real(z.modulus * z.modulus)
}

/// The modulus of the conjugate of a product is the product of the moduli:
/// |conj(z·w)| = |z|·|w|.
theorem complex_modulus_conj_product(a: Complex, b: Complex) {
    (a * b).conj.modulus = a.modulus * b.modulus
} by {
    complex_conj_product(a, b)
    (a * b).conj = a.conj * b.conj
    (a * b).conj.modulus = (a.conj * b.conj).modulus
    modulus_mul(a.conj, b.conj)
    (a.conj * b.conj).modulus = a.conj.modulus * b.conj.modulus
    (a * b).conj.modulus = a.conj.modulus * b.conj.modulus
    complex_modulus_conj(a)
    a.conj.modulus = a.modulus
    complex_modulus_conj(b)
    b.conj.modulus = b.modulus
    (a * b).conj.modulus = a.modulus * b.modulus
}

// ---------------------------------------------------------------------------
// e. The inverse and division laws.
// ---------------------------------------------------------------------------

/// A nonzero complex number multiplied by its inverse is one:
/// z·z^(-1) = 1 for z != 0.
theorem complex_mul_inverse_self(a: Complex) {
    a != Complex.0 implies a * a.inverse = Complex.1
} by {
    if a != Complex.0 {
        a * a.inverse = Complex.1
    }
}

/// The inverse of a product is the product of the inverses:
/// (z·w)^(-1) = z^(-1)·w^(-1).
theorem complex_inverse_mul(a: Complex, b: Complex) {
    (a * b).inverse = a.inverse * b.inverse
} by {
    inverse_dist[Complex](a, b)
    (a * b).inverse = a.inverse * b.inverse
}

/// A nonzero complex number divided by itself is one: z / z = 1 for z != 0.
theorem complex_div_self(a: Complex) {
    a != Complex.0 implies a / a = Complex.1
} by {
    div_self(a)
    a != Complex.0 implies a / a = Complex.1
}

/// The inverse of a nonzero complex number is nonzero.
theorem complex_inverse_nonzero(a: Complex) {
    a != Complex.0 implies a.inverse != Complex.0
} by {
    if a != Complex.0 {
        a * a.inverse = Complex.1
        if a.inverse = Complex.0 {
            a * a.inverse = a * Complex.0
            mul_zero_right(a)
            a * Complex.0 = Complex.0
            a * a.inverse = Complex.0
            Complex.1 = Complex.0
            zero_is_different_than_one
            Complex.0 != Complex.1
            false
        }
        a.inverse != Complex.0
    }
}

/// The modulus of the inverse is the inverse of the modulus:
/// |z^(-1)| = |z|^(-1) for z != 0.
theorem complex_modulus_inverse(a: Complex) {
    a != Complex.0 implies a.inverse.modulus = a.modulus.inverse
} by {
    if a != Complex.0 {
        if a.modulus = Real.0 {
            modulus_eq_zero(a)
            a.modulus = Real.0 iff a = Complex.0
            a = Complex.0
            false
        }
        a.modulus != Real.0
        a * a.inverse = Complex.1
        modulus_mul(a, a.inverse)
        (a * a.inverse).modulus = a.modulus * a.inverse.modulus
        modulus_of_one
        Complex.1.modulus = Real.1
        (a * a.inverse).modulus = Complex.1.modulus
        (a * a.inverse).modulus = Real.1
        a.modulus * a.inverse.modulus = Real.1
        a.modulus * a.modulus.inverse = Real.1
        real_mul_left_cancel(a.modulus, a.inverse.modulus, a.modulus.inverse)
        a.modulus != Real.0 and a.modulus * a.inverse.modulus = a.modulus * a.modulus.inverse implies a.inverse.modulus = a.modulus.inverse
        a.inverse.modulus = a.modulus.inverse
    }
}

/// Conjugation commutes with inversion: conj(z^(-1)) = conj(z)^(-1).
theorem complex_conj_inverse_restated(a: Complex) {
    a.inverse.conj = a.conj.inverse
} by {
    conj_inverse(a)
    a.inverse.conj = a.conj.inverse
}

/// Conjugation distributes over division: conj(z / w) = conj(z) / conj(w).
theorem complex_conj_div_restated(a: Complex, b: Complex) {
    (a / b).conj = a.conj / b.conj
} by {
    conj_div(a, b)
    (a / b).conj = a.conj / b.conj
}

/// The modulus of a quotient is the quotient of the moduli:
/// |z / w| = |z| / |w| for w != 0.
theorem complex_modulus_div(a: Complex, b: Complex) {
    b != Complex.0 implies (a / b).modulus = a.modulus / b.modulus
} by {
    if b != Complex.0 {
        a / b = a * b.inverse
        (a / b).modulus = (a * b.inverse).modulus
        modulus_mul(a, b.inverse)
        (a * b.inverse).modulus = a.modulus * b.inverse.modulus
        (a / b).modulus = a.modulus * b.inverse.modulus
        complex_modulus_inverse(b)
        b != Complex.0 implies b.inverse.modulus = b.modulus.inverse
        b.inverse.modulus = b.modulus.inverse
        (a / b).modulus = a.modulus * b.modulus.inverse
        a.modulus * b.modulus.inverse = a.modulus / b.modulus
        (a / b).modulus = a.modulus / b.modulus
    }
}

// ---------------------------------------------------------------------------
// f. The exponential identities and Euler's formula.
// ---------------------------------------------------------------------------

/// The exponential of the negative is the multiplicative inverse:
/// z.exp·(-z).exp = 1.
theorem complex_exp_mul_neg(z: Complex) {
    complex_exp(z) * complex_exp(-z) = Complex.1
} by {
    complex_exp_add(z, -z)
    complex_exp(z + -z) = complex_exp(z) * complex_exp(-z)
    add_neg_cancel(z)
    z + -z = Complex.0
    complex_exp(z + -z) = complex_exp(Complex.0)
    complex_exp_zero
    complex_exp(Complex.0) = Complex.1
    complex_exp(z + -z) = Complex.1
    complex_exp(z) * complex_exp(-z) = Complex.1
}

/// The complex exponential never vanishes: z.exp != 0 for every z.
theorem complex_exp_nonzero(z: Complex) {
    complex_exp(z) != Complex.0
} by {
    complex_exp_mul_neg(z)
    complex_exp(z) * complex_exp(-z) = Complex.1
    if complex_exp(z) = Complex.0 {
        complex_exp(z) * complex_exp(-z) = Complex.0 * complex_exp(-z)
        mul_zero_left(complex_exp(-z))
        Complex.0 * complex_exp(-z) = Complex.0
        complex_exp(z) * complex_exp(-z) = Complex.0
        Complex.1 = Complex.0
        zero_is_different_than_one
        Complex.0 != Complex.1
        false
    }
    complex_exp(z) != Complex.0
}

/// The exponential of the negative is the inverse of the exponential:
/// (-z).exp = z.exp^(-1).
theorem complex_exp_neg(z: Complex) {
    complex_exp(-z) = complex_exp(z).inverse
} by {
    complex_exp_mul_neg(z)
    complex_exp(z) * complex_exp(-z) = Complex.1
    complex_exp_nonzero(z)
    complex_exp(z) != Complex.0
    complex_exp(z) * complex_exp(z).inverse = Complex.1
    mul_left_cancel(complex_exp(z), complex_exp(-z), complex_exp(z).inverse)
    complex_exp(z) * complex_exp(-z) = complex_exp(z) * complex_exp(z).inverse and complex_exp(z) != Complex.0 implies complex_exp(-z) = complex_exp(z).inverse
    complex_exp(-z) = complex_exp(z).inverse
}

/// The exponential of a difference is the quotient of exponentials:
/// (z - w).exp = z.exp·(-w).exp.
theorem complex_exp_sub(z: Complex, w: Complex) {
    complex_exp(z - w) = complex_exp(z) * complex_exp(-w)
} by {
    z - w = z + -w
    complex_exp(z - w) = complex_exp(z + -w)
    complex_exp_add(z, -w)
    complex_exp(z + -w) = complex_exp(z) * complex_exp(-w)
    complex_exp(z - w) = complex_exp(z) * complex_exp(-w)
}

/// The exponential of a difference in inverse form:
/// (z - w).exp = z.exp·w.exp^(-1).
theorem complex_exp_div(z: Complex, w: Complex) {
    complex_exp(z - w) = complex_exp(z) * complex_exp(w).inverse
} by {
    complex_exp_sub(z, w)
    complex_exp(z - w) = complex_exp(z) * complex_exp(-w)
    complex_exp_neg(w)
    complex_exp(-w) = complex_exp(w).inverse
    complex_exp(z - w) = complex_exp(z) * complex_exp(w).inverse
}

/// The complex exponential lifts the real addition law:
/// (x + y).exp = x.exp·y.exp for embedded real x and y.
theorem complex_exp_from_real_add(x: Real, y: Real) {
    complex_exp(Complex.from_real(x + y)) =
        complex_exp(Complex.from_real(x)) * complex_exp(Complex.from_real(y))
} by {
    real_add_lifts(x, y)
    Complex.from_real(x + y) = Complex.from_real(x) + Complex.from_real(y)
    complex_exp(Complex.from_real(x + y)) =
        complex_exp(Complex.from_real(x) + Complex.from_real(y))
    complex_exp_add(Complex.from_real(x), Complex.from_real(y))
    complex_exp(Complex.from_real(x) + Complex.from_real(y)) =
        complex_exp(Complex.from_real(x)) * complex_exp(Complex.from_real(y))
    complex_exp(Complex.from_real(x + y)) =
        complex_exp(Complex.from_real(x)) * complex_exp(Complex.from_real(y))
}

/// Euler's identity: (i·pi).exp = -1.
theorem complex_euler_identity {
    complex_exp(Complex.i * Complex.from_real(pi)) = -Complex.1
} by {
    euler_identity
    complex_exp(Complex.i * Complex.from_real(pi)) = -Complex.1
}

/// Euler's formula for an embedded real number:
/// (i·x).exp = x.cos + i·x.sin.
theorem complex_euler_formula_embedded(x: Real) {
    complex_exp(Complex.i * Complex.from_real(x)) =
        Complex.from_real(x.cos) + Complex.i * Complex.from_real(x.sin)
} by {
    complex_i_mul_real(x) = Complex.i * Complex.from_real(x)
    complex_exp(Complex.i * Complex.from_real(x)) = complex_exp(complex_i_mul_real(x))
    complex_exp_i_mul_real_eq(x)
    complex_exp(complex_i_mul_real(x)) = Complex.new(x.cos, x.sin)
    complex_exp(Complex.i * Complex.from_real(x)) = Complex.new(x.cos, x.sin)
    Complex.new(x.cos, x.sin) = Complex.from_real(x.cos) + Complex.i * Complex.from_real(x.sin)
    complex_exp(Complex.i * Complex.from_real(x)) =
        Complex.from_real(x.cos) + Complex.i * Complex.from_real(x.sin)
}

/// The exponential of 2·pi·i is one: (2·pi·i).exp = 1.
theorem complex_exp_two_pi_i {
    complex_exp(Complex.i * Complex.from_real(two * pi)) = Complex.1
} by {
    complex_i_mul_real(two * pi) = Complex.i * Complex.from_real(two * pi)
    complex_exp(Complex.i * Complex.from_real(two * pi)) = complex_exp(complex_i_mul_real(two * pi))
    complex_exp_two_pi
    complex_exp(complex_i_mul_real(two * pi)) = Complex.1
    complex_exp(Complex.i * Complex.from_real(two * pi)) = Complex.1
}

/// The complex exponential satisfies the functional equation:
/// (z + w).exp = z.exp·w.exp.
theorem complex_exp_add_restated(z: Complex, w: Complex) {
    complex_exp(z + w) = complex_exp(z) * complex_exp(w)
} by {
    complex_exp_add(z, w)
    complex_exp(z + w) = complex_exp(z) * complex_exp(w)
}

/// The modulus of the exponential of an embedded real number is the
/// exponential itself: |x.exp| = x.exp for every real x.
theorem complex_exp_real_modulus(x: Real) {
    complex_exp(Complex.from_real(x)).modulus = x.exp
} by {
    complex_exp_from_real(x)
    complex_exp(Complex.from_real(x)) = Complex.from_real(x.exp)
    complex_exp(Complex.from_real(x)).modulus = Complex.from_real(x.exp).modulus
    modulus_from_real(x.exp)
    Complex.from_real(x.exp).modulus = x.exp.abs
    complex_exp(Complex.from_real(x)).modulus = x.exp.abs
    exp_pos(x)
    x.exp > Real.0
    x.exp.is_positive
    pos_imp_eq_abs(x.exp)
    x.exp = x.exp.abs
    complex_exp(Complex.from_real(x)).modulus = x.exp
}

/// The complex logarithm of a product of exponentials of embedded real numbers
/// is the sum of the arguments: (x.exp·y.exp).log = x + y.
theorem complex_log_exp_sum(x: Real, y: Real) {
    complex_log(complex_exp(Complex.from_real(x)) * complex_exp(Complex.from_real(y))) =
        Complex.from_real(x + y)
} by {
    complex_exp_from_real_add(x, y)
    complex_exp(Complex.from_real(x + y)) =
        complex_exp(Complex.from_real(x)) * complex_exp(Complex.from_real(y))
    complex_log(complex_exp(Complex.from_real(x)) * complex_exp(Complex.from_real(y))) =
        complex_log(complex_exp(Complex.from_real(x + y)))
    complex_log_exp_real(x + y)
    complex_log(complex_exp(Complex.from_real(x + y))) = Complex.from_real(x + y)
    complex_log(complex_exp(Complex.from_real(x)) * complex_exp(Complex.from_real(y))) =
        Complex.from_real(x + y)
}

