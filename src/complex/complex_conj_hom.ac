from complex.complex import Complex, conj_add, conj_mul, conj_one, conj_conj
from algebra.ring.ring_hom import RingHom, is_ring_hom, ring_hom_eq_of_hom_eq,
    identity_ring_hom, identity_ring_hom_hom, compose_ring_hom, compose_ring_hom_hom
from data.basic.functions import identity_fn, compose

/// The complex conjugate as a function on complex numbers.
let complex_conj_fn: Complex -> Complex = function(c: Complex) {
    c.conj
}

/// Complex conjugation preserves addition pointwise.
theorem complex_conj_fn_add(a: Complex, b: Complex) {
    complex_conj_fn(a + b) = complex_conj_fn(a) + complex_conj_fn(b)
} by {
    conj_add(a, b)
}

/// Complex conjugation preserves multiplication pointwise.
theorem complex_conj_fn_mul(a: Complex, b: Complex) {
    complex_conj_fn(a * b) = complex_conj_fn(a) * complex_conj_fn(b)
} by {
    conj_mul(a, b)
}

/// Complex conjugation sends one to one.
theorem complex_conj_fn_one {
    complex_conj_fn(Complex.1) = Complex.1
} by {
    conj_one
}

/// Complex conjugation is an involution: applying it twice yields the identity.
theorem complex_conj_fn_involution(a: Complex) {
    complex_conj_fn(complex_conj_fn(a)) = a
} by {
    conj_conj(a)
}

/// Complex conjugation preserves the ring operations and the multiplicative identity.
theorem complex_conj_fn_is_ring_hom {
    is_ring_hom(complex_conj_fn)
} by {
    complex_conj_fn_one
    forall(a: Complex, b: Complex) {
        complex_conj_fn_add(a, b)
        complex_conj_fn_mul(a, b)
        complex_conj_fn(a + b) = complex_conj_fn(a) + complex_conj_fn(b) and
            complex_conj_fn(a * b) = complex_conj_fn(a) * complex_conj_fn(b)
    }
}

/// `RingHom.new` produces a `Some` value for complex conjugation.
theorem complex_conj_fn_ring_hom_some {
    exists(h: RingHom[Complex, Complex]) {
        RingHom.new(complex_conj_fn) = Option.some(h)
    }
} by {
    complex_conj_fn_is_ring_hom
}

/// Complex conjugation as a ring homomorphism of the complex numbers.
let complex_conj_ring_hom: RingHom[Complex, Complex] satisfy {
    RingHom.new(complex_conj_fn) = Option.some(complex_conj_ring_hom)
}

/// The underlying function of the complex conjugation ring homomorphism is complex conjugation.
theorem complex_conj_ring_hom_hom {
    complex_conj_ring_hom.hom = complex_conj_fn
} by {
}

/// Composing the complex conjugation function with itself gives the identity function.
theorem complex_conj_fn_compose_self {
    compose(complex_conj_fn, complex_conj_fn) = identity_fn[Complex]
} by {
    forall(a: Complex) {
        complex_conj_fn_involution(a)
        compose(complex_conj_fn, complex_conj_fn)(a) = identity_fn[Complex](a)
    }
}

/// The composite of the complex conjugation ring homomorphism with itself has
/// the identity function as its underlying map.
theorem complex_conj_ring_hom_compose_self_hom {
    compose_ring_hom(complex_conj_ring_hom, complex_conj_ring_hom).hom = identity_fn[Complex]
} by {
    compose_ring_hom_hom(complex_conj_ring_hom, complex_conj_ring_hom)
    complex_conj_ring_hom_hom
    complex_conj_fn_compose_self
}

/// The complex conjugation ring homomorphism is involutive under composition.
theorem complex_conj_ring_hom_involutive {
    compose_ring_hom(complex_conj_ring_hom, complex_conj_ring_hom) = identity_ring_hom[Complex]
} by {
    let lhs = compose_ring_hom(complex_conj_ring_hom, complex_conj_ring_hom)
    let rhs = identity_ring_hom[Complex]
    complex_conj_ring_hom_compose_self_hom
    lhs.hom = identity_fn[Complex]
    identity_ring_hom_hom[Complex]
    rhs.hom = identity_fn[Complex]
    lhs.hom = rhs.hom
    ring_hom_eq_of_hom_eq(lhs, rhs)
}
