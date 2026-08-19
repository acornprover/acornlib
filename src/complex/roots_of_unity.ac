from nat import Nat, pow_zero, one_pow, from_nat, from_nat_add, from_nat_mul, from_nat_zero,
    from_nat_one, alt_induction, zero_or_suc, only_zero_lte_zero, alt_suc_ne_zero
from real import Real, two, two_nonzero, div_mul_cancel_left, mul_div_cancel, div_cancel_common,
    from_nat_suc_pos_real, pi, pi_over_two, real_mul_comm
from complex.complex import Complex, distrib, mul_comm, mul_assoc, real_add_lifts, real_mul_lifts,
    from_real_one, from_real_zero, mul_zero_left, mul_one_right
from complex.complex_exp import complex_exp, complex_i_mul_real, complex_exp_add, euler_identity,
    complex_exp_zero, complex_pow_suc
from complex.complex_trig import real_two_mul, complex_from_real_two_nonzero

/// The primitive n-th root of unity: omega_n = exp(2·pi·i / n).
define omega(n: Nat) -> Complex {
    complex_exp(complex_i_mul_real(two * pi / from_nat[Real](n)))
}

/// Multiplication by i distributes over real addition:
/// i·(x + y) = i·x + i·y.
theorem complex_i_mul_real_add(x: Real, y: Real) {
    complex_i_mul_real(x + y) = complex_i_mul_real(x) + complex_i_mul_real(y)
} by {
    complex_i_mul_real(x + y) = Complex.i * Complex.from_real(x + y)
    real_add_lifts(x, y)
    Complex.from_real(x + y) = Complex.from_real(x) + Complex.from_real(y)
    Complex.i * Complex.from_real(x + y) = Complex.i * (Complex.from_real(x) + Complex.from_real(y))
    distrib(Complex.i, Complex.from_real(x), Complex.from_real(y))
    Complex.i * (Complex.from_real(x) + Complex.from_real(y)) =
        Complex.i * Complex.from_real(x) + Complex.i * Complex.from_real(y)
    complex_i_mul_real(x) = Complex.i * Complex.from_real(x)
    complex_i_mul_real(y) = Complex.i * Complex.from_real(y)
    Complex.i * Complex.from_real(x) + Complex.i * Complex.from_real(y) =
        complex_i_mul_real(x) + complex_i_mul_real(y)
    complex_i_mul_real(x + y) = complex_i_mul_real(x) + complex_i_mul_real(y)
}

/// A real scalar commutes with multiplication by i:
/// r·(i·x) = i·(r·x).
theorem complex_i_mul_real_mul(r: Real, x: Real) {
    Complex.from_real(r) * complex_i_mul_real(x) = complex_i_mul_real(r * x)
} by {
    complex_i_mul_real(x) = Complex.i * Complex.from_real(x)
    Complex.from_real(r) * complex_i_mul_real(x) = Complex.from_real(r) * (Complex.i * Complex.from_real(x))
    mul_assoc(Complex.from_real(r), Complex.i, Complex.from_real(x))
    Complex.from_real(r) * (Complex.i * Complex.from_real(x)) =
        (Complex.from_real(r) * Complex.i) * Complex.from_real(x)
    mul_comm(Complex.from_real(r), Complex.i)
    Complex.from_real(r) * Complex.i = Complex.i * Complex.from_real(r)
    (Complex.from_real(r) * Complex.i) * Complex.from_real(x) =
        (Complex.i * Complex.from_real(r)) * Complex.from_real(x)
    mul_assoc(Complex.i, Complex.from_real(r), Complex.from_real(x))
    (Complex.i * Complex.from_real(r)) * Complex.from_real(x) =
        Complex.i * (Complex.from_real(r) * Complex.from_real(x))
    real_mul_lifts(r, x)
    Complex.from_real(r * x) = Complex.from_real(r) * Complex.from_real(x)
    Complex.i * (Complex.from_real(r) * Complex.from_real(x)) = Complex.i * Complex.from_real(r * x)
    complex_i_mul_real(r * x) = Complex.i * Complex.from_real(r * x)
    Complex.from_real(r) * complex_i_mul_real(x) = complex_i_mul_real(r * x)
}

/// Natural-number powers of the complex exponential satisfy the functional
/// equation: exp(z)^n = exp(n·z).
theorem complex_exp_pow(z: Complex, n: Nat) {
    complex_exp(z).pow(n) = complex_exp(Complex.from_real(from_nat[Real](n)) * z)
} by {
    define p(k: Nat) -> Bool {
        complex_exp(z).pow(k) = complex_exp(Complex.from_real(from_nat[Real](k)) * z)
    }
    pow_zero(complex_exp(z))
    complex_exp(z).pow(Nat.0) = Complex.1
    from_nat_zero[Real]
    from_nat[Real](Nat.0) = Real.0
    from_real_zero
    Complex.from_real(Real.0) = Complex.0
    Complex.from_real(from_nat[Real](Nat.0)) * z = Complex.0 * z
    mul_zero_left(z)
    Complex.0 * z = Complex.0
    complex_exp_zero
    complex_exp(Complex.0) = Complex.1
    complex_exp(Complex.from_real(from_nat[Real](Nat.0)) * z) = Complex.1
    p(Nat.0)
    forall(k: Nat) {
        if p(k) {
            complex_pow_suc(complex_exp(z), k)
            complex_exp(z).pow(k.suc) = complex_exp(z) * complex_exp(z).pow(k)
            complex_exp(z).pow(k) = complex_exp(Complex.from_real(from_nat[Real](k)) * z)
            complex_exp(z) * complex_exp(z).pow(k) =
                complex_exp(z) * complex_exp(Complex.from_real(from_nat[Real](k)) * z)
            complex_exp_add(Complex.from_real(from_nat[Real](k)) * z, z)
            complex_exp(Complex.from_real(from_nat[Real](k)) * z + z) =
                complex_exp(Complex.from_real(from_nat[Real](k)) * z) * complex_exp(z)
            from_nat[Real](k.suc) = from_nat[Real](k) + Real.1
            real_add_lifts(from_nat[Real](k), Real.1)
            Complex.from_real(from_nat[Real](k) + Real.1) =
                Complex.from_real(from_nat[Real](k)) + Complex.from_real(Real.1)
            from_real_one
            Complex.from_real(Real.1) = Complex.1
            Complex.from_real(from_nat[Real](k.suc)) =
                Complex.from_real(from_nat[Real](k)) + Complex.1
            Complex.from_real(from_nat[Real](k.suc)) * z =
                (Complex.from_real(from_nat[Real](k)) + Complex.1) * z
            distrib(z, Complex.from_real(from_nat[Real](k)), Complex.1)
            z * (Complex.from_real(from_nat[Real](k)) + Complex.1) =
                z * Complex.from_real(from_nat[Real](k)) + z * Complex.1
            mul_comm(z, Complex.from_real(from_nat[Real](k)) + Complex.1)
            (Complex.from_real(from_nat[Real](k)) + Complex.1) * z =
                z * (Complex.from_real(from_nat[Real](k)) + Complex.1)
            (Complex.from_real(from_nat[Real](k)) + Complex.1) * z =
                z * Complex.from_real(from_nat[Real](k)) + z * Complex.1
            mul_comm(z, Complex.from_real(from_nat[Real](k)))
            z * Complex.from_real(from_nat[Real](k)) =
                Complex.from_real(from_nat[Real](k)) * z
            mul_one_right(z)
            z * Complex.1 = z
            (Complex.from_real(from_nat[Real](k)) + Complex.1) * z =
                Complex.from_real(from_nat[Real](k)) * z + z
            Complex.from_real(from_nat[Real](k.suc)) * z =
                Complex.from_real(from_nat[Real](k)) * z + z
            complex_exp(Complex.from_real(from_nat[Real](k.suc)) * z) =
                complex_exp(Complex.from_real(from_nat[Real](k)) * z + z)
            complex_exp(Complex.from_real(from_nat[Real](k.suc)) * z) =
                complex_exp(Complex.from_real(from_nat[Real](k)) * z) * complex_exp(z)
            complex_exp(z).pow(k.suc) =
                complex_exp(Complex.from_real(from_nat[Real](k.suc)) * z)
            p(k.suc)
        }
    }
    p(Nat.0) and forall(k: Nat) { p(k) implies p(k.suc) }
    alt_induction(p)
    p(n)
}

/// The exponential of two pi times i is one: exp(2·pi·i) = 1.
theorem complex_exp_two_pi {
    complex_exp(complex_i_mul_real(two * pi)) = Complex.1
} by {
    real_two_mul(pi)
    two * pi = pi + pi
    complex_i_mul_real(two * pi) = complex_i_mul_real(pi + pi)
    complex_i_mul_real_add(pi, pi)
    complex_i_mul_real(pi + pi) = complex_i_mul_real(pi) + complex_i_mul_real(pi)
    complex_i_mul_real(two * pi) = complex_i_mul_real(pi) + complex_i_mul_real(pi)
    complex_exp_add(complex_i_mul_real(pi), complex_i_mul_real(pi))
    complex_exp(complex_i_mul_real(pi) + complex_i_mul_real(pi)) =
        complex_exp(complex_i_mul_real(pi)) * complex_exp(complex_i_mul_real(pi))
    complex_exp(complex_i_mul_real(two * pi)) =
        complex_exp(complex_i_mul_real(pi)) * complex_exp(complex_i_mul_real(pi))
    euler_identity
    complex_exp(Complex.i * Complex.from_real(pi)) = -Complex.1
    complex_i_mul_real(pi) = Complex.i * Complex.from_real(pi)
    complex_exp(complex_i_mul_real(pi)) = -Complex.1
    (-Complex.1) * (-Complex.1) = Complex.1
    complex_exp(complex_i_mul_real(two * pi)) = Complex.1
}

/// A natural number at least one is nonzero.
theorem nat_ge_one_ne_zero(n: Nat) {
    n >= Nat.1 implies n != Nat.0
} by {
    if n >= Nat.1 {
        if n = Nat.0 {
            Nat.1 <= Nat.0
            only_zero_lte_zero(Nat.1)
            Nat.1 = Nat.0
            Nat.0.suc = Nat.1
            Nat.0.suc = Nat.0
            alt_suc_ne_zero(Nat.0)
            Nat.0.suc != Nat.0
            false
        }
        n != Nat.0
    }
}

/// The n-th root of unity raised to the n-th power is one:
/// (omega_n)^n = 1 for every positive n.
theorem omega_pow(n: Nat) {
    n >= Nat.1 implies omega(n).pow(n) = Complex.1
} by {
    if n >= Nat.1 {
        omega(n) = complex_exp(complex_i_mul_real(two * pi / from_nat[Real](n)))
        omega(n).pow(n) = complex_exp(complex_i_mul_real(two * pi / from_nat[Real](n))).pow(n)
        complex_exp_pow(complex_i_mul_real(two * pi / from_nat[Real](n)), n)
        complex_exp(complex_i_mul_real(two * pi / from_nat[Real](n))).pow(n) =
            complex_exp(Complex.from_real(from_nat[Real](n)) * complex_i_mul_real(two * pi / from_nat[Real](n)))
        complex_i_mul_real_mul(from_nat[Real](n), two * pi / from_nat[Real](n))
        Complex.from_real(from_nat[Real](n)) * complex_i_mul_real(two * pi / from_nat[Real](n)) =
            complex_i_mul_real(from_nat[Real](n) * (two * pi / from_nat[Real](n)))
        complex_exp(Complex.from_real(from_nat[Real](n)) * complex_i_mul_real(two * pi / from_nat[Real](n))) =
            complex_exp(complex_i_mul_real(from_nat[Real](n) * (two * pi / from_nat[Real](n))))
        nat_ge_one_ne_zero(n)
        n >= Nat.1 implies n != Nat.0
        n != Nat.0
        zero_or_suc(n)
        let k: Nat satisfy {
            n = k.suc
        }
        from_nat_suc_pos_real(k)
        from_nat[Real](k.suc) > Real.0
        from_nat[Real](n) > Real.0
        from_nat[Real](n) != Real.0
        mul_div_cancel(two * pi, from_nat[Real](n))
        from_nat[Real](n) * (two * pi / from_nat[Real](n)) = two * pi
        complex_exp(complex_i_mul_real(from_nat[Real](n) * (two * pi / from_nat[Real](n)))) =
            complex_exp(complex_i_mul_real(two * pi))
        complex_exp_two_pi
        complex_exp(complex_i_mul_real(two * pi)) = Complex.1
        omega(n).pow(n) = Complex.1
    }
}

/// Natural-number powers add exponents: z^(m + n) = z^m · z^n.
theorem complex_pow_add(z: Complex, m: Nat, n: Nat) {
    z.pow(m + n) = z.pow(m) * z.pow(n)
} by {
    define p(k: Nat) -> Bool {
        z.pow(m + k) = z.pow(m) * z.pow(k)
    }
    z.pow(m + Nat.0) = z.pow(m)
    pow_zero(z)
    z.pow(Nat.0) = Complex.1
    mul_one_right(z.pow(m))
    z.pow(m) * Complex.1 = z.pow(m)
    z.pow(m) * z.pow(Nat.0) = z.pow(m)
    z.pow(m + Nat.0) = z.pow(m) * z.pow(Nat.0)
    p(Nat.0)
    forall(k: Nat) {
        if p(k) {
            z.pow(m + k) = z.pow(m) * z.pow(k)
            complex_pow_suc(z, m + k)
            z.pow((m + k).suc) = z * z.pow(m + k)
            (m + k).suc = m + k.suc
            z.pow(m + k.suc) = z * z.pow(m + k)
            z.pow(m + k.suc) = z * (z.pow(m) * z.pow(k))
            z * (z.pow(m) * z.pow(k)) = z.pow(m) * (z * z.pow(k))
            complex_pow_suc(z, k)
            z.pow(k.suc) = z * z.pow(k)
            z.pow(m) * z.pow(k.suc) = z.pow(m) * (z * z.pow(k))
            z.pow(m + k.suc) = z.pow(m) * z.pow(k.suc)
            p(k.suc)
        }
    }
    p(Nat.0) and forall(k: Nat) { p(k) implies p(k.suc) }
    alt_induction(p)
    p(n)
}

/// Natural-number powers compose: (z^m)^n = z^(m·n).
theorem complex_pow_pow(z: Complex, m: Nat, n: Nat) {
    z.pow(m).pow(n) = z.pow(m * n)
} by {
    define p(k: Nat) -> Bool {
        z.pow(m).pow(k) = z.pow(m * k)
    }
    pow_zero(z.pow(m))
    z.pow(m).pow(Nat.0) = Complex.1
    pow_zero(z)
    z.pow(Nat.0) = Complex.1
    m * Nat.0 = Nat.0
    z.pow(m * Nat.0) = z.pow(Nat.0)
    z.pow(m * Nat.0) = Complex.1
    z.pow(m).pow(Nat.0) = z.pow(m * Nat.0)
    p(Nat.0)
    forall(k: Nat) {
        if p(k) {
            complex_pow_suc(z.pow(m), k)
            z.pow(m).pow(k.suc) = z.pow(m) * z.pow(m).pow(k)
            z.pow(m).pow(k) = z.pow(m * k)
            z.pow(m).pow(k.suc) = z.pow(m) * z.pow(m * k)
            complex_pow_add(z, m, m * k)
            z.pow(m + m * k) = z.pow(m) * z.pow(m * k)
            z.pow(m).pow(k.suc) = z.pow(m + m * k)
            m + m * k = m * k.suc
            z.pow(m * k.suc) = z.pow(m + m * k)
            z.pow(m).pow(k.suc) = z.pow(m * k.suc)
            p(k.suc)
        }
    }
    p(Nat.0) and forall(k: Nat) { p(k) implies p(k.suc) }
    alt_induction(p)
    p(n)
}

/// The natural number two, embedded in the reals, is the real number two.
theorem from_nat_two_real {
    from_nat[Real](Nat.2) = two
} by {
    from_nat_add[Real](Nat.1, Nat.1)
    from_nat[Real](Nat.1 + Nat.1) = from_nat[Real](Nat.1) + from_nat[Real](Nat.1)
    Nat.1 + Nat.1 = Nat.2
    from_nat[Real](Nat.2) = from_nat[Real](Nat.1) + from_nat[Real](Nat.1)
    from_nat_one[Real]
    from_nat[Real](Nat.1) = Real.1
    from_nat[Real](Nat.2) = Real.1 + Real.1
    two = Real.1 + Real.1
    from_nat[Real](Nat.2) = two
}

/// The second root of unity is negative one: omega_2 = -1.
theorem omega_two_neg_one {
    omega(Nat.2) = -Complex.1
} by {
    omega(Nat.2) = complex_exp(complex_i_mul_real(two * pi / from_nat[Real](Nat.2)))
    from_nat_two_real
    from_nat[Real](Nat.2) = two
    two * pi / from_nat[Real](Nat.2) = two * pi / two
    two_nonzero
    two != Real.0
    div_mul_cancel_left(two, pi)
    (two * pi) / two = pi
    two * pi / from_nat[Real](Nat.2) = pi
    complex_i_mul_real(two * pi / from_nat[Real](Nat.2)) = complex_i_mul_real(pi)
    complex_exp(complex_i_mul_real(two * pi / from_nat[Real](Nat.2))) =
        complex_exp(complex_i_mul_real(pi))
    euler_identity
    complex_exp(Complex.i * Complex.from_real(pi)) = -Complex.1
    complex_i_mul_real(pi) = Complex.i * Complex.from_real(pi)
    complex_exp(complex_i_mul_real(pi)) = -Complex.1
    omega(Nat.2) = -Complex.1
}

/// Two pi over four is pi over two.
theorem two_pi_over_four_eq {
    two * pi / from_nat[Real](Nat.4) = pi / two
} by {
    from_nat_two_real
    from_nat[Real](Nat.2) = two
    from_nat_mul[Real](Nat.2, Nat.2)
    from_nat[Real](Nat.2 * Nat.2) = from_nat[Real](Nat.2) * from_nat[Real](Nat.2)
    Nat.2 * Nat.2 = Nat.4
    from_nat[Real](Nat.4) = from_nat[Real](Nat.2) * from_nat[Real](Nat.2)
    from_nat[Real](Nat.4) = two * two
    two * pi / from_nat[Real](Nat.4) = two * pi / (two * two)
    real_mul_comm(pi, two)
    pi * two = two * pi
    two_nonzero
    two != Real.0
    div_cancel_common(pi, two, two)
    (pi * two) / (two * two) = pi / two
    (two * pi) / (two * two) = pi / two
    two * pi / from_nat[Real](Nat.4) = pi / two
}

/// The square of the fourth root of unity is negative one: omega_4^2 = -1.
theorem omega_four_sq_neg_one {
    omega(Nat.4).pow(Nat.2) = -Complex.1
} by {
    omega(Nat.4) = complex_exp(complex_i_mul_real(two * pi / from_nat[Real](Nat.4)))
    omega(Nat.4).pow(Nat.2) = complex_exp(complex_i_mul_real(two * pi / from_nat[Real](Nat.4))).pow(Nat.2)
    complex_exp_pow(complex_i_mul_real(two * pi / from_nat[Real](Nat.4)), Nat.2)
    complex_exp(complex_i_mul_real(two * pi / from_nat[Real](Nat.4))).pow(Nat.2) =
        complex_exp(Complex.from_real(from_nat[Real](Nat.2)) * complex_i_mul_real(two * pi / from_nat[Real](Nat.4)))
    complex_i_mul_real_mul(from_nat[Real](Nat.2), two * pi / from_nat[Real](Nat.4))
    Complex.from_real(from_nat[Real](Nat.2)) * complex_i_mul_real(two * pi / from_nat[Real](Nat.4)) =
        complex_i_mul_real(from_nat[Real](Nat.2) * (two * pi / from_nat[Real](Nat.4)))
    complex_exp(Complex.from_real(from_nat[Real](Nat.2)) * complex_i_mul_real(two * pi / from_nat[Real](Nat.4))) =
        complex_exp(complex_i_mul_real(from_nat[Real](Nat.2) * (two * pi / from_nat[Real](Nat.4))))

    two_pi_over_four_eq
    two * pi / from_nat[Real](Nat.4) = pi / two
    from_nat_two_real
    from_nat[Real](Nat.2) = two
    from_nat[Real](Nat.2) * (two * pi / from_nat[Real](Nat.4)) = two * (pi / two)
    two_nonzero
    two != Real.0
    mul_div_cancel(pi, two)
    two * (pi / two) = pi
    from_nat[Real](Nat.2) * (two * pi / from_nat[Real](Nat.4)) = pi

    complex_exp(complex_i_mul_real(from_nat[Real](Nat.2) * (two * pi / from_nat[Real](Nat.4)))) =
        complex_exp(complex_i_mul_real(pi))
    euler_identity
    complex_exp(Complex.i * Complex.from_real(pi)) = -Complex.1
    complex_i_mul_real(pi) = Complex.i * Complex.from_real(pi)
    complex_exp(complex_i_mul_real(pi)) = -Complex.1
    omega(Nat.4).pow(Nat.2) = -Complex.1
}

/// Pi over two is the constant pi_over_two.
theorem pi_div_two_eq {
    pi / two = pi_over_two
} by {
    pi = two * pi_over_two
    two_nonzero
    two != Real.0
    div_mul_cancel_left(two, pi_over_two)
    (two * pi_over_two) / two = pi_over_two
    pi / two = pi_over_two
}

/// The fourth root of unity is exp(i·pi/2).
theorem omega_four_exp {
    omega(Nat.4) = complex_exp(complex_i_mul_real(pi / two))
} by {
    omega(Nat.4) = complex_exp(complex_i_mul_real(two * pi / from_nat[Real](Nat.4)))
    two_pi_over_four_eq
    two * pi / from_nat[Real](Nat.4) = pi / two
    complex_i_mul_real(two * pi / from_nat[Real](Nat.4)) = complex_i_mul_real(pi / two)
    complex_exp(complex_i_mul_real(two * pi / from_nat[Real](Nat.4))) =
        complex_exp(complex_i_mul_real(pi / two))
    omega(Nat.4) = complex_exp(complex_i_mul_real(pi / two))
}

// The fourth root of unity is the imaginary unit: omega_4 = i.
// This is not yet proved.  Writing omega_4 = exp(i·pi/2) (omega_four_exp) and
// using Euler's formula exp(i·x) = x.cos + i·x.sin (proved as
// complex_euler_formula in complex_trig.ac), the claim reduces to
// (pi/2).cos = 0 and (pi/2).sin = 1.  Both are proved in real/pi.ac
// (cos_pi_over_two_zero, sin_pi_over_two_one) but are not exported by the
// public 'real' module, and re-deriving them needs the mean value theorem.
// With those two facts:
//   omega_4 = exp(i·pi/2) = (pi/2).cos + i·(pi/2).sin = 0 + i = i.
// theorem omega_four_i {
//     omega(Nat.4) = Complex.i
// }

// The n powers omega_n^0, ..., omega_n^(n-1) are pairwise distinct.
// Not yet proved: omega_n^j = omega_n^k with 0 <= j < k < n would give
// exp(2·pi·i·(j-k)/n) = 1 with a nonzero exponent strictly between -2·pi·i
// and 2·pi·i, so distinctness needs the fact that exp(i·x) = 1 only when x
// is an integer multiple of 2·pi (injectivity of exp on a fundamental strip),
// which is not available in the current library.
// theorem omega_pow_distinct(n: Nat, j: Nat, k: Nat) {
//     n >= Nat.1 and j < k and k < n implies omega(n).pow(j) != omega(n).pow(k)
// }

/// The fourth root of unity is not one.
theorem omega_four_ne_one {
    omega(Nat.4) != Complex.1
} by {
    if omega(Nat.4) = Complex.1 {
        omega(Nat.4).pow(Nat.2) = Complex.1.pow(Nat.2)
        Complex.1.pow(Nat.2) = Complex.1
        omega_four_sq_neg_one
        omega(Nat.4).pow(Nat.2) = -Complex.1
        Complex.1 = -Complex.1
        Complex.1 + Complex.1 = -Complex.1 + Complex.1
        -Complex.1 + Complex.1 = Complex.0
        Complex.1 + Complex.1 = Complex.0
        Complex.1 + Complex.1 = Complex.from_real(Real.1 + Real.1)
        Real.1 + Real.1 = two
        Complex.from_real(Real.1 + Real.1) = Complex.from_real(two)
        Complex.from_real(two) = Complex.0
        complex_from_real_two_nonzero
        Complex.from_real(two) != Complex.0
        false
    }
    omega(Nat.4) != Complex.1
}

/// The fourth root of unity is not negative one.
theorem omega_four_ne_neg_one {
    omega(Nat.4) != -Complex.1
} by {
    if omega(Nat.4) = -Complex.1 {
        omega(Nat.4).pow(Nat.2) = (-Complex.1).pow(Nat.2)
        (-Complex.1).pow(Nat.2) = Complex.1
        omega_four_sq_neg_one
        omega(Nat.4).pow(Nat.2) = -Complex.1
        Complex.1 = -Complex.1
        Complex.1 + Complex.1 = -Complex.1 + Complex.1
        -Complex.1 + Complex.1 = Complex.0
        Complex.1 + Complex.1 = Complex.0
        Complex.1 + Complex.1 = Complex.from_real(Real.1 + Real.1)
        Real.1 + Real.1 = two
        Complex.from_real(Real.1 + Real.1) = Complex.from_real(two)
        Complex.from_real(two) = Complex.0
        complex_from_real_two_nonzero
        Complex.from_real(two) != Complex.0
        false
    }
    omega(Nat.4) != -Complex.1
}

/// A complex number z is an n-th root of unity when z^n = 1.
define is_root_of_unity(z: Complex, n: Nat) -> Bool {
    z.pow(n) = Complex.1
}

/// Every power of the primitive n-th root of unity is an n-th root of unity.
theorem omega_pow_is_root_of_unity(n: Nat, k: Nat) {
    n >= Nat.1 implies is_root_of_unity(omega(n).pow(k), n)
} by {
    if n >= Nat.1 {
        complex_pow_pow(omega(n), k, n)
        omega(n).pow(k).pow(n) = omega(n).pow(k * n)
        k * n = n * k
        omega(n).pow(k * n) = omega(n).pow(n * k)
        complex_pow_pow(omega(n), n, k)
        omega(n).pow(n).pow(k) = omega(n).pow(n * k)
        omega(n).pow(k * n) = omega(n).pow(n).pow(k)
        omega_pow(n)
        n >= Nat.1 implies omega(n).pow(n) = Complex.1
        omega(n).pow(n) = Complex.1
        one_pow[Complex](k)
        Complex.1.pow(k) = Complex.1
        omega(n).pow(n).pow(k) = Complex.1
        omega(n).pow(k).pow(n) = Complex.1
        is_root_of_unity(omega(n).pow(k), n)
    }
}
