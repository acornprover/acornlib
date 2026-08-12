// ---------------------------------------------------------------------------
// The powers of the imaginary unit and related small powers.
//
// This module collects the values and periodicities of the natural powers of
// i and -1, and the small algebraic identities around i:
//
//   a. i³ = -i, (-i)² = -1, (-1)³ = -1, and the periodicity i^(n+4) = i^n.
//   b. i·conj(i) = 1.
//   c. The factorizations (1 + i)·(1 - i) = 2 and (1 + i)² = 2·i.
// ---------------------------------------------------------------------------

from nat import Nat
from real import Real, two
from complex.complex import Complex, mul_one_right, mul_one_left, real_add_lifts, from_real_one,
    i_squared_eq_neg_one, add_assoc, add_comm, add_neg_cancel, add_zero_left
from complex.complex_algebra_deep import neg_mul_right, neg_neg_complex, sub_neg_eq_add,
    two_eq_real_one_plus_one, diff_of_squares, square_of_sum, conj_i
from complex.complex_exp import complex_pow_suc
from complex.complex_properties import pow_two_i, pow_two_neg_one, pow_two_neg, pow_four_i
from complex.complex_trig_deep import complex_two_mul
from complex.roots_of_unity import complex_pow_add

/// The cube of the imaginary unit is -i: i³ = -i.
theorem pow_three_i {
    Complex.i.pow(Nat.3) = -Complex.i
} by {
    complex_pow_suc(Complex.i, Nat.2)
    Complex.i.pow(Nat.2.suc) = Complex.i * Complex.i.pow(Nat.2)
    Nat.2.suc = Nat.3
    Complex.i.pow(Nat.3) = Complex.i * Complex.i.pow(Nat.2)
    pow_two_i
    Complex.i.pow(Nat.2) = -Complex.1
    Complex.i.pow(Nat.3) = Complex.i * (-Complex.1)
    neg_mul_right(Complex.i, Complex.1)
    Complex.i * (-Complex.1) = -(Complex.i * Complex.1)
    mul_one_right(Complex.i)
    Complex.i * Complex.1 = Complex.i
    Complex.i * (-Complex.1) = -Complex.i
    Complex.i.pow(Nat.3) = -Complex.i
}

/// The powers of i are periodic with period four: i^(n+4) = i^n.
theorem i_pow_add_four(n: Nat) {
    Complex.i.pow(n + Nat.4) = Complex.i.pow(n)
} by {
    complex_pow_add(Complex.i, n, Nat.4)
    Complex.i.pow(n + Nat.4) = Complex.i.pow(n) * Complex.i.pow(Nat.4)
    pow_four_i
    Complex.i.pow(Nat.4) = Complex.1
    Complex.i.pow(n) * Complex.i.pow(Nat.4) = Complex.i.pow(n) * Complex.1
    mul_one_right(Complex.i.pow(n))
    Complex.i.pow(n) * Complex.1 = Complex.i.pow(n)
    Complex.i.pow(n + Nat.4) = Complex.i.pow(n)
}

/// The square of -i is -1: (-i)² = -1.
theorem pow_two_neg_i {
    (-Complex.i).pow(Nat.2) = -Complex.1
} by {
    pow_two_neg(Complex.i)
    (-Complex.i).pow(Nat.2) = Complex.i.pow(Nat.2)
    pow_two_i
    Complex.i.pow(Nat.2) = -Complex.1
    (-Complex.i).pow(Nat.2) = -Complex.1
}

/// The cube of -1 is -1: (-1)³ = -1.
theorem pow_three_neg_one {
    (-Complex.1).pow(Nat.3) = -Complex.1
} by {
    complex_pow_suc(-Complex.1, Nat.2)
    (-Complex.1).pow(Nat.2.suc) = (-Complex.1) * (-Complex.1).pow(Nat.2)
    Nat.2.suc = Nat.3
    (-Complex.1).pow(Nat.3) = (-Complex.1) * (-Complex.1).pow(Nat.2)
    pow_two_neg_one
    (-Complex.1).pow(Nat.2) = Complex.1
    (-Complex.1).pow(Nat.3) = (-Complex.1) * Complex.1
    mul_one_right(-Complex.1)
    (-Complex.1) * Complex.1 = -Complex.1
    (-Complex.1).pow(Nat.3) = -Complex.1
}

/// The product of i with its conjugate is one: i·conj(i) = 1.
theorem i_mul_conj_i {
    Complex.i * Complex.i.conj = Complex.1
} by {
    conj_i
    Complex.i.conj = -Complex.i
    Complex.i * Complex.i.conj = Complex.i * (-Complex.i)
    neg_mul_right(Complex.i, Complex.i)
    Complex.i * (-Complex.i) = -(Complex.i * Complex.i)
    i_squared_eq_neg_one
    Complex.i * Complex.i = -Complex.1
    -(Complex.i * Complex.i) = -(-Complex.1)
    neg_neg_complex(Complex.1)
    -(-Complex.1) = Complex.1
    Complex.i * Complex.i.conj = Complex.1
}

/// The factorization (1 + i)·(1 - i) = 2.
theorem one_plus_i_times_one_minus_i {
    (Complex.1 + Complex.i) * (Complex.1 - Complex.i) = Complex.from_real(two)
} by {
    diff_of_squares(Complex.1, Complex.i)
    (Complex.1 + Complex.i) * (Complex.1 - Complex.i) = Complex.1 * Complex.1 - Complex.i * Complex.i
    mul_one_left(Complex.1)
    Complex.1 * Complex.1 = Complex.1
    i_squared_eq_neg_one
    Complex.i * Complex.i = -Complex.1
    Complex.1 * Complex.1 - Complex.i * Complex.i = Complex.1 - (-Complex.1)
    sub_neg_eq_add(Complex.1, Complex.1)
    Complex.1 - (-Complex.1) = Complex.1 + Complex.1
    Complex.1 + Complex.1 = Complex.from_real(Real.1 + Real.1)
    two_eq_real_one_plus_one
    two = Real.1 + Real.1
    Complex.from_real(Real.1 + Real.1) = Complex.from_real(two)
    Complex.1 + Complex.1 = Complex.from_real(two)
    (Complex.1 + Complex.i) * (Complex.1 - Complex.i) = Complex.from_real(two)
}

/// The square of 1 + i is 2·i: (1 + i)² = 2·i.
theorem one_plus_i_sq {
    (Complex.1 + Complex.i) * (Complex.1 + Complex.i) = Complex.from_real(two) * Complex.i
} by {
    square_of_sum(Complex.1, Complex.i)
    (Complex.1 + Complex.i) * (Complex.1 + Complex.i) =
        Complex.1 * Complex.1 + (Complex.1 * Complex.i + Complex.1 * Complex.i) + Complex.i * Complex.i
    mul_one_left(Complex.1)
    Complex.1 * Complex.1 = Complex.1
    mul_one_left(Complex.i)
    Complex.1 * Complex.i = Complex.i
    Complex.1 * Complex.i + Complex.1 * Complex.i = Complex.i + Complex.i
    complex_two_mul(Complex.i)
    Complex.from_real(two) * Complex.i = Complex.i + Complex.i
    i_squared_eq_neg_one
    Complex.i * Complex.i = -Complex.1
    add_assoc(Complex.1, Complex.from_real(two) * Complex.i, -Complex.1)
    Complex.1 + Complex.from_real(two) * Complex.i + (-Complex.1) =
        Complex.1 + (Complex.from_real(two) * Complex.i + (-Complex.1))
    add_comm(Complex.from_real(two) * Complex.i, -Complex.1)
    Complex.from_real(two) * Complex.i + (-Complex.1) = (-Complex.1) + Complex.from_real(two) * Complex.i
    Complex.1 + (Complex.from_real(two) * Complex.i + (-Complex.1)) =
        Complex.1 + ((-Complex.1) + Complex.from_real(two) * Complex.i)
    add_assoc(Complex.1, -Complex.1, Complex.from_real(two) * Complex.i)
    Complex.1 + ((-Complex.1) + Complex.from_real(two) * Complex.i) =
        (Complex.1 + -Complex.1) + Complex.from_real(two) * Complex.i
    add_neg_cancel(Complex.1)
    Complex.1 + -Complex.1 = Complex.0
    (Complex.1 + -Complex.1) + Complex.from_real(two) * Complex.i =
        Complex.0 + Complex.from_real(two) * Complex.i
    add_zero_left(Complex.from_real(two) * Complex.i)
    Complex.0 + Complex.from_real(two) * Complex.i = Complex.from_real(two) * Complex.i
    Complex.1 + Complex.from_real(two) * Complex.i + (-Complex.1) =
        Complex.from_real(two) * Complex.i
    (Complex.1 + Complex.i) * (Complex.1 + Complex.i) = Complex.from_real(two) * Complex.i
}

