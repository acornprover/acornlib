// ---------------------------------------------------------------------------
// Additional laws of the complex logarithm.
//
// This module extends src/complex/complex_log_deep.ac with:
//
//   a. Conjugation commutes with the logarithm: (conj(z)).log = conj(z.log).
//   b. The product law (z·w).log = z.log + w.log for nonzero z and w, which
//      holds because the logarithm is real-valued and the moduli multiply.
//   c. The restricted inverse law (z.log).exp = |z| on the punctured plane,
//      and injectivity of the complex exponential on the real axis.
//   d. The exponential is periodic with every integer multiple of 2·pi·i:
//      (n·2·pi·i).exp = 1.
// ---------------------------------------------------------------------------

from real import Real, two, pi, exp_injective, log_mul, log_some_of_pos_exists, exp_log_or_zero, mul_pos_pos, exp_pos
from nat import Nat, from_nat, one_pow
from complex.complex import Complex, real_add_lifts, re_from_real, from_real_zero, from_real_one, conj_from_real
from complex.complex_abs import modulus_mul, modulus_conj
from complex.complex_abs_deep import modulus_pos
from complex.complex_algebra_deep import complex_eq_new_components
from complex.complex_exp import complex_exp, complex_exp_from_real, complex_i_mul_real
from complex.complex_exp_deep import complex_exp_nat_mul
from complex.complex_log import complex_log
from complex.roots_of_unity import complex_exp_two_pi, complex_i_mul_real_mul

/// A real complex number equals the embedding of its real part.
theorem complex_real_eq_from_real(z: Complex) {
    z.is_real implies z = Complex.from_real(z.re)
} by {
    if z.is_real {
        z.im = Real.0
        complex_eq_new_components(z)
        z = Complex.new(z.re, z.im)
        z = Complex.new(z.re, Real.0)
        Complex.from_real(z.re) = Complex.new(z.re, Real.0)
        z = Complex.from_real(z.re)
    }
}

/// Conjugation commutes with the complex logarithm:
/// conj(z.log) = (conj(z)).log.
theorem complex_log_conj(z: Complex) {
    complex_log(z).conj = complex_log(z.conj)
} by {
    complex_log(z) = Complex.from_real((z.modulus).log.get_or_else(Real.0))
    complex_log(z).conj = Complex.from_real((z.modulus).log.get_or_else(Real.0)).conj
    conj_from_real((z.modulus).log.get_or_else(Real.0))
    Complex.from_real((z.modulus).log.get_or_else(Real.0)).conj = Complex.from_real((z.modulus).log.get_or_else(Real.0))
    complex_log(z).conj = Complex.from_real((z.modulus).log.get_or_else(Real.0))
    complex_log(z.conj) = Complex.from_real((z.conj.modulus).log.get_or_else(Real.0))
    modulus_conj(z)
    z.conj.modulus = z.modulus
    (z.conj.modulus).log.get_or_else(Real.0) = (z.modulus).log.get_or_else(Real.0)
    Complex.from_real((z.conj.modulus).log.get_or_else(Real.0)) = Complex.from_real((z.modulus).log.get_or_else(Real.0))
    complex_log(z.conj) = Complex.from_real((z.modulus).log.get_or_else(Real.0))
    complex_log(z).conj = complex_log(z.conj)
}

/// The product law of the complex logarithm:
/// (z·w).log = z.log + w.log for nonzero z and w.
theorem complex_log_mul(z: Complex, w: Complex) {
    z != Complex.0 and w != Complex.0 implies complex_log(z * w) = complex_log(z) + complex_log(w)
} by {
    if z != Complex.0 and w != Complex.0 {
        complex_log(z * w) = Complex.from_real(((z * w).modulus).log.get_or_else(Real.0))
        modulus_mul(z, w)
        (z * w).modulus = z.modulus * w.modulus
        ((z * w).modulus).log.get_or_else(Real.0) = (z.modulus * w.modulus).log.get_or_else(Real.0)
        modulus_pos(z)
        z != Complex.0 implies z.modulus > Real.0
        z.modulus > Real.0
        z.modulus.is_positive
        modulus_pos(w)
        w != Complex.0 implies w.modulus > Real.0
        w.modulus > Real.0
        w.modulus.is_positive
        mul_pos_pos(z.modulus, w.modulus)
        z.modulus.is_positive and w.modulus.is_positive implies (z.modulus * w.modulus).is_positive
        (z.modulus * w.modulus).is_positive
        (z.modulus * w.modulus) > Real.0
        log_some_of_pos_exists(z.modulus)
        let a: Real satisfy {
            z.modulus.log = Option.some(a)
        }
        log_some_of_pos_exists(w.modulus)
        let b: Real satisfy {
            w.modulus.log = Option.some(b)
        }
        log_mul(z.modulus, w.modulus, a, b)
        z.modulus > Real.0 and w.modulus > Real.0 and z.modulus.log = Option.some(a) and w.modulus.log = Option.some(b) implies (z.modulus * w.modulus).log = Option.some(a + b)
        (z.modulus * w.modulus).log = Option.some(a + b)
        (z.modulus).log.get_or_else(Real.0) = option_get_or_else(z.modulus.log, Real.0)
        option_get_or_else_some[Real](a, Real.0)
        option_get_or_else(Option.some(a), Real.0) = a
        (z.modulus).log.get_or_else(Real.0) = a
        (w.modulus).log.get_or_else(Real.0) = option_get_or_else(w.modulus.log, Real.0)
        option_get_or_else_some[Real](b, Real.0)
        option_get_or_else(Option.some(b), Real.0) = b
        (w.modulus).log.get_or_else(Real.0) = b
        (z.modulus * w.modulus).log.get_or_else(Real.0) = option_get_or_else((z.modulus * w.modulus).log, Real.0)
        option_get_or_else_some[Real](a + b, Real.0)
        option_get_or_else(Option.some(a + b), Real.0) = a + b
        (z.modulus * w.modulus).log.get_or_else(Real.0) = a + b
        (z.modulus * w.modulus).log.get_or_else(Real.0) = (z.modulus).log.get_or_else(Real.0) + (w.modulus).log.get_or_else(Real.0)
        Complex.from_real(((z * w).modulus).log.get_or_else(Real.0)) =
            Complex.from_real((z.modulus).log.get_or_else(Real.0) + (w.modulus).log.get_or_else(Real.0))
        real_add_lifts((z.modulus).log.get_or_else(Real.0), (w.modulus).log.get_or_else(Real.0))
        Complex.from_real((z.modulus).log.get_or_else(Real.0) + (w.modulus).log.get_or_else(Real.0)) =
            Complex.from_real((z.modulus).log.get_or_else(Real.0)) + Complex.from_real((w.modulus).log.get_or_else(Real.0))
        complex_log(z) = Complex.from_real((z.modulus).log.get_or_else(Real.0))
        complex_log(w) = Complex.from_real((w.modulus).log.get_or_else(Real.0))
        complex_log(z * w) = complex_log(z) + complex_log(w)
    }
}

/// The exponential of the logarithm is the modulus on the punctured plane:
/// (z.log).exp = |z| for z != 0.
theorem complex_exp_log_modulus(z: Complex) {
    z != Complex.0 implies complex_exp(complex_log(z)) = Complex.from_real(z.modulus)
} by {
    if z != Complex.0 {
        complex_log(z) = Complex.from_real((z.modulus).log.get_or_else(Real.0))
        complex_exp(complex_log(z)) = complex_exp(Complex.from_real((z.modulus).log.get_or_else(Real.0)))
        complex_exp_from_real((z.modulus).log.get_or_else(Real.0))
        complex_exp(Complex.from_real((z.modulus).log.get_or_else(Real.0))) = Complex.from_real(((z.modulus).log.get_or_else(Real.0)).exp)
        modulus_pos(z)
        z != Complex.0 implies z.modulus > Real.0
        z.modulus > Real.0
        log_some_of_pos_exists(z.modulus)
        let y: Real satisfy {
            z.modulus.log = Option.some(y)
        }
        exp_log_or_zero(z.modulus, y)
        z.modulus > Real.0 and z.modulus.log = Option.some(y) implies y.exp = z.modulus
        y.exp = z.modulus
        (z.modulus).log.get_or_else(Real.0) = option_get_or_else(z.modulus.log, Real.0)
        option_get_or_else_some[Real](y, Real.0)
        option_get_or_else(Option.some(y), Real.0) = y
        (z.modulus).log.get_or_else(Real.0) = y
        ((z.modulus).log.get_or_else(Real.0)).exp = y.exp
        ((z.modulus).log.get_or_else(Real.0)).exp = z.modulus
        Complex.from_real(((z.modulus).log.get_or_else(Real.0)).exp) = Complex.from_real(z.modulus)
        complex_exp(complex_log(z)) = Complex.from_real(z.modulus)
    }
}

/// The complex exponential is injective on the real axis.
theorem complex_exp_injective_real(z: Complex, w: Complex) {
    complex_exp(z) = complex_exp(w) and z.is_real and w.is_real implies z = w
} by {
    if complex_exp(z) = complex_exp(w) and z.is_real and w.is_real {
        complex_real_eq_from_real(z)
        z = Complex.from_real(z.re)
        complex_real_eq_from_real(w)
        w = Complex.from_real(w.re)
        complex_exp_from_real(z.re)
        complex_exp(Complex.from_real(z.re)) = Complex.from_real((z.re).exp)
        complex_exp(z) = Complex.from_real((z.re).exp)
        complex_exp_from_real(w.re)
        complex_exp(Complex.from_real(w.re)) = Complex.from_real((w.re).exp)
        complex_exp(w) = Complex.from_real((w.re).exp)
        Complex.from_real((z.re).exp) = Complex.from_real((w.re).exp)
        re_from_real((z.re).exp)
        Complex.from_real((z.re).exp).re = (z.re).exp
        re_from_real((w.re).exp)
        Complex.from_real((w.re).exp).re = (w.re).exp
        (z.re).exp = (w.re).exp
        exp_injective(z.re, w.re)
        (z.re).exp = (w.re).exp implies z.re = w.re
        z.re = w.re
        Complex.from_real(z.re) = Complex.from_real(w.re)
        z = w
    }
}

/// The exponential is periodic with every integer multiple of 2·pi·i:
/// (n·2·pi·i).exp = 1.
theorem complex_exp_periodic_int(n: Nat) {
    complex_exp(complex_i_mul_real(from_nat[Real](n) * (two * pi))) = Complex.1
} by {
    complex_exp_nat_mul(complex_i_mul_real(two * pi), n)
    complex_exp(Complex.from_real(from_nat[Real](n)) * complex_i_mul_real(two * pi)) =
        complex_exp(complex_i_mul_real(two * pi)).pow(n)
    complex_i_mul_real_mul(from_nat[Real](n), two * pi)
    Complex.from_real(from_nat[Real](n)) * complex_i_mul_real(two * pi) =
        complex_i_mul_real(from_nat[Real](n) * (two * pi))
    complex_exp(complex_i_mul_real(from_nat[Real](n) * (two * pi))) =
        complex_exp(complex_i_mul_real(two * pi)).pow(n)
    complex_exp_two_pi
    complex_exp(complex_i_mul_real(two * pi)) = Complex.1
    complex_exp(complex_i_mul_real(two * pi)).pow(n) = Complex.1.pow(n)
    one_pow[Complex](n)
    Complex.1.pow(n) = Complex.1
    complex_exp(complex_i_mul_real(from_nat[Real](n) * (two * pi))) = Complex.1
}

