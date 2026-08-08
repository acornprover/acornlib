from real import Real
from algebra.add_group import AddGroupHom, is_add_group_hom
from complex.complex import Complex, complex_re_fn, complex_im_fn, complex_re_fn_add, complex_im_fn_add

/// `complex_re_fn` is an additive group homomorphism.
theorem complex_re_fn_is_hom {
    is_add_group_hom(complex_re_fn)
} by {
    forall(a: Complex, b: Complex) {
        complex_re_fn_add(a, b)
        complex_re_fn(a + b) = complex_re_fn(a) + complex_re_fn(b)
    }
}

/// `complex_im_fn` is an additive group homomorphism.
theorem complex_im_fn_is_hom {
    is_add_group_hom(complex_im_fn)
} by {
    forall(a: Complex, b: Complex) {
        complex_im_fn_add(a, b)
        complex_im_fn(a + b) = complex_im_fn(a) + complex_im_fn(b)
    }
}

/// The real-part function packaged as an additive group homomorphism.
let complex_re_hom: AddGroupHom[Complex, Real] satisfy {
    AddGroupHom.new(complex_re_fn) = Option.some(complex_re_hom)
}

/// The imaginary-part function packaged as an additive group homomorphism.
let complex_im_hom: AddGroupHom[Complex, Real] satisfy {
    AddGroupHom.new(complex_im_fn) = Option.some(complex_im_hom)
}

/// The underlying function of `complex_re_hom` is `complex_re_fn`.
theorem complex_re_hom_hom {
    complex_re_hom.hom = complex_re_fn
}

/// The underlying function of `complex_im_hom` is `complex_im_fn`.
theorem complex_im_hom_hom {
    complex_im_hom.hom = complex_im_fn
}
