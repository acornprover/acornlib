from real import Real
from complex.complex import Complex, complex_real_smul, re_smul, im_smul, conj_from_real
from complex.complex_conj_hom import complex_conj_fn, complex_conj_fn_add

/// Complex conjugation preserves real scalar multiplication on the complex numbers:
/// `(r · z).conj = r · z.conj` where `r · z = complex_real_smul(r, z)`.
theorem complex_conj_fn_real_smul(r: Real, x: Complex) {
    complex_conj_fn(complex_real_smul(r, x)) = complex_real_smul(r, complex_conj_fn(x))
} by {
    let sx: Complex = complex_real_smul(r, x)
    re_smul(r, x)
    im_smul(r, x)
    sx.conj = Complex.new(r * x.re, r * (-x.im))
    let scx: Complex = complex_real_smul(r, x.conj)
    scx = Complex.new(r * x.re, r * (-x.im))
}

/// Complex conjugation sends zero to zero.
theorem complex_conj_fn_zero {
    complex_conj_fn(Complex.0) = Complex.0
} by {
}

/// Complex conjugation commutes with negation.
theorem complex_conj_fn_neg(x: Complex) {
    complex_conj_fn(-x) = -complex_conj_fn(x)
} by {
    -x.conj = Complex.new(-x.re, -(-x.im))
}


/// Complex conjugation commutes with subtraction.
theorem complex_conj_fn_sub(x: Complex, y: Complex) {
    complex_conj_fn(x - y) = complex_conj_fn(x) - complex_conj_fn(y)
} by {
    complex_conj_fn_add(x, -y)
    complex_conj_fn(x - y) = complex_conj_fn(x) + complex_conj_fn(-y)
    complex_conj_fn_neg(y)
}

/// Complex conjugation fixes images of the real-to-complex embedding.
theorem complex_conj_fn_from_real(a: Real) {
    complex_conj_fn(Complex.from_real(a)) = Complex.from_real(a)
} by {
    conj_from_real(a)
}

