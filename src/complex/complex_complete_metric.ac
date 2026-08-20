from nat import Nat
from order import lte_lt_trans, lte_trans, lte_max_left, lte_max_right
from real import Real
from real import converges_to
from real import add_lt_lt, lt_trans, eps_lt_half
from real import is_cauchy_seq, cauchy_imp_exists_limit
from analysis import is_cauchy_metric, cauchy_metric_bound, cauchy_metric_bound_at,
    cauchy_metric_bound_from_distances, tendsto_metric, CompleteMetricSpace,
    real_distance, real_is_cauchy_metric_imp_is_cauchy_seq
from complex.complex import re_sub, im_sub
from complex.complex_abs import re_abs_le_modulus, im_abs_le_modulus, modulus_le_re_abs_add_im_abs
from complex.complex_seq import re_seq, im_seq, componentwise_imp_complex_converges_to
from complex.complex_metric import Complex, complex_distance_eq_modulus_sub, complex_converges_to_imp_tendsto_metric

/// A complex metric-Cauchy sequence has a metric-Cauchy real-part sequence.
theorem complex_is_cauchy_metric_imp_re_is_cauchy_metric(q: Nat -> Complex) {
    is_cauchy_metric(q) implies is_cauchy_metric(re_seq(q))
} by {
    if is_cauchy_metric(q) {
        forall(eps: Real) {
            if eps.is_positive {
                let n: Nat satisfy {
                    cauchy_metric_bound(q, n, eps)
                }
                forall(i: Nat, j: Nat) {
                    if n <= i and n <= j {
                        cauchy_metric_bound_at(q, n, eps, i, j)
                        complex_distance_eq_modulus_sub(q(i), q(j))
                        re_abs_le_modulus(q(i) - q(j))
                        lte_lt_trans(((q(i) - q(j)).re).abs, (q(i) - q(j)).modulus, eps)
                        ((q(i) - q(j)).re).abs < eps
                        re_sub(q(i), q(j))
                        re_seq(q, i) = q(i).re
                        re_seq(q, j) = q(j).re
                        real_distance(re_seq(q, i), re_seq(q, j)) = (re_seq(q, i) - re_seq(q, j)).abs
                        re_seq(q, i).distance(re_seq(q, j)) = real_distance(re_seq(q, i), re_seq(q, j))
                        re_seq(q, i).distance(re_seq(q, j)) < eps
                    }
                }
                cauchy_metric_bound_from_distances(re_seq(q), n, eps)
                exists(k: Nat) {
                    cauchy_metric_bound(re_seq(q), k, eps)
                }
            }
        }
    }
}

/// A complex metric-Cauchy sequence has a metric-Cauchy imaginary-part sequence.
theorem complex_is_cauchy_metric_imp_im_is_cauchy_metric(q: Nat -> Complex) {
    is_cauchy_metric(q) implies is_cauchy_metric(im_seq(q))
} by {
    if is_cauchy_metric(q) {
        forall(eps: Real) {
            if eps.is_positive {
                let n: Nat satisfy {
                    cauchy_metric_bound(q, n, eps)
                }
                forall(i: Nat, j: Nat) {
                    if n <= i and n <= j {
                        cauchy_metric_bound_at(q, n, eps, i, j)
                        complex_distance_eq_modulus_sub(q(i), q(j))
                        im_abs_le_modulus(q(i) - q(j))
                        lte_lt_trans(((q(i) - q(j)).im).abs, (q(i) - q(j)).modulus, eps)
                        ((q(i) - q(j)).im).abs < eps
                        im_sub(q(i), q(j))
                        im_seq(q, i) = q(i).im
                        im_seq(q, j) = q(j).im
                        real_distance(im_seq(q, i), im_seq(q, j)) = (im_seq(q, i) - im_seq(q, j)).abs
                        im_seq(q, i).distance(im_seq(q, j)) = real_distance(im_seq(q, i), im_seq(q, j))
                        im_seq(q, i).distance(im_seq(q, j)) < eps
                    }
                }
                cauchy_metric_bound_from_distances(im_seq(q), n, eps)
                exists(k: Nat) {
                    cauchy_metric_bound(im_seq(q), k, eps)
                }
            }
        }
    }
}

/// Componentwise metric-Cauchy real and imaginary parts make a complex sequence metric-Cauchy.
theorem componentwise_cauchy_metric_imp_complex_is_cauchy_metric(q: Nat -> Complex) {
    is_cauchy_metric(re_seq(q)) and is_cauchy_metric(im_seq(q)) implies is_cauchy_metric(q)
} by {
    if is_cauchy_metric(re_seq(q)) and is_cauchy_metric(im_seq(q)) {
        forall(eps: Real) {
            if eps.is_positive {
                eps_lt_half(eps)
                let half: Real satisfy {
                    half.is_positive and half + half < eps
                }
                let nr: Nat satisfy {
                    cauchy_metric_bound(re_seq(q), nr, half)
                }
                (forall(e: Real) {
                    e.is_positive implies exists(n: Nat) {
                        cauchy_metric_bound(im_seq(q), n, e)
                    }
                }) = true
                let ni: Nat satisfy {
                    cauchy_metric_bound(im_seq(q), ni, half)
                }
                let n = nr.max(ni)
                lte_max_left(nr, ni)
                lte_max_right(nr, ni)
                forall(i: Nat, j: Nat) {
                    if n <= i and n <= j {
                        lte_trans(nr, n, i)
                        lte_trans(nr, n, j)
                        nr <= j
                        lte_trans(ni, n, i)
                        lte_trans(ni, n, j)
                        ni <= j

                        cauchy_metric_bound_at(re_seq(q), nr, half, i, j)
                        re_seq(q, i).distance(re_seq(q, j)) < half
                        re_seq(q, i).distance(re_seq(q, j)) = real_distance(re_seq(q, i), re_seq(q, j))
                        re_sub(q(i), q(j))
                        re_seq(q, i) = q(i).re
                        re_seq(q, j) = q(j).re
                        ((q(i) - q(j)).re).abs < half

                        cauchy_metric_bound_at(im_seq(q), ni, half, i, j)
                        im_seq(q, i).distance(im_seq(q, j)) < half
                        im_seq(q, i).distance(im_seq(q, j)) = real_distance(im_seq(q, i), im_seq(q, j))
                        im_sub(q(i), q(j))
                        im_seq(q, i) = q(i).im
                        im_seq(q, j) = q(j).im
                        ((q(i) - q(j)).im).abs < half

                        modulus_le_re_abs_add_im_abs(q(i) - q(j))
                        add_lt_lt(((q(i) - q(j)).re).abs, half, ((q(i) - q(j)).im).abs, half)
                        lte_lt_trans((q(i) - q(j)).modulus, ((q(i) - q(j)).re).abs + ((q(i) - q(j)).im).abs, half + half)
                        lt_trans((q(i) - q(j)).modulus, half + half, eps)
                        complex_distance_eq_modulus_sub(q(i), q(j))
                        q(i).distance(q(j)) < eps
                    }
                }
                forall(i: Nat, j: Nat) {
                    n <= i and n <= j implies q(i).distance(q(j)) < eps
                }
                cauchy_metric_bound[Complex](q, n, eps) = forall(i: Nat, j: Nat) {
                    n <= i and n <= j implies q(i).distance(q(j)) < eps
                }
                if not cauchy_metric_bound[Complex](q, n, eps) {
                    let w0: Nat satisfy {
                        exists(k0: Nat) {
                            n <= w0 and n <= k0 and not q(w0).distance(q(k0)) < eps
                        }
                    }
                    let w1: Nat satisfy {
                        n <= w0 and n <= w1 and not q(w0).distance(q(w1)) < eps
                    }
                    let w2: Nat satisfy {
                        n <= w0 and n <= w2 and not q(w0).distance(q(w2)) < eps
                    }
                    false
                }
                exists(k: Nat) {
                    cauchy_metric_bound(q, k, eps)
                }
            }
        }
    }
}

/// A complex sequence is metric-Cauchy exactly when its real and imaginary part sequences are metric-Cauchy.
theorem complex_is_cauchy_metric_iff_components(q: Nat -> Complex) {
    is_cauchy_metric(q) = (is_cauchy_metric(re_seq(q)) and is_cauchy_metric(im_seq(q)))
} by {
    if is_cauchy_metric(q) {
        complex_is_cauchy_metric_imp_re_is_cauchy_metric(q)
        complex_is_cauchy_metric_imp_im_is_cauchy_metric(q)
        is_cauchy_metric(im_seq(q))
        is_cauchy_metric(re_seq(q)) and is_cauchy_metric(im_seq(q))
    }
    if is_cauchy_metric(re_seq(q)) and is_cauchy_metric(im_seq(q)) {
        componentwise_cauchy_metric_imp_complex_is_cauchy_metric(q)
        is_cauchy_metric(q)
    }
}

/// Every complex metric-Cauchy sequence has a metric limit.
theorem complex_cauchy_metric_imp_tendsto(q: Nat -> Complex) {
    is_cauchy_metric(q) implies exists(a: Complex) {
        tendsto_metric(q, a)
    }
} by {
    if is_cauchy_metric(q) {
        complex_is_cauchy_metric_imp_re_is_cauchy_metric(q)
        real_is_cauchy_metric_imp_is_cauchy_seq(re_seq(q))
        cauchy_imp_exists_limit(re_seq(q))
        let a_re: Real satisfy {
            converges_to(re_seq(q), a_re)
        }
        complex_is_cauchy_metric_imp_im_is_cauchy_metric(q)
        real_is_cauchy_metric_imp_is_cauchy_seq(im_seq(q))
        cauchy_imp_exists_limit(im_seq(q))
        let a_im: Real satisfy {
            converges_to(im_seq(q), a_im)
        }
        let a = Complex.new(a_re, a_im)
        converges_to(re_seq(q), a.re)
        converges_to(im_seq(q), a.im)
        componentwise_imp_complex_converges_to(q, a)
        complex_converges_to_imp_tendsto_metric(q, a)
        exists(w: Complex) {
            tendsto_metric(q, w)
        }
    }
}

/// The complex numbers form a complete metric space.
instance Complex: CompleteMetricSpace
