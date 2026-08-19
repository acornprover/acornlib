// ---------------------------------------------------------------------------
// Deepening of the complex exponential theory.
//
// This module extends src/complex/complex_exp.ac with the structural
// identities of the complex exponential:
//
//   a. Conjugation commutes with the exponential:
//      conj(z.exp) = (conj(z)).exp, proved by pushing conjugation through
//      the series limit and through each term z^n / n!.
//   b. The modulus of the exponential: |z.exp| = (Re(z)).exp, and as a
//      consequence |(i·x).exp| = 1 for every real x.
//   c. The Cartesian decomposition z.exp = (Re(z)).exp·(i·Im(z)).exp and
//      z.exp = (Re(z)).exp·((Im(z)).cos + i·(Im(z)).sin).
//   d. The remaining exponential laws: (z - w).exp = z.exp/w.exp,
//      (n·z).exp = z.exp^n, and (z + i·pi).exp = -z.exp.
// ---------------------------------------------------------------------------

from real import Real, pi, exp_add, exp_zero, exp_pos, cos_pi_neg_one, sin_pi_zero
from nat import Nat, from_nat, alt_induction
from rat import Rat
from list import partial
from data.basic.functions import function_extensionality
from complex.complex import Complex, conj_add, conj_div, conj_from_real, conj_zero, re_from_real, im_from_real, re_mul, mul_one_right, real_mul_lifts, add_zero_right, eq_by_components
from complex.complex_abs import modulus_nonneg, nonneg_eq_of_squares_eq
from complex.complex_algebra_deep import z_add_conj, complex_eq_new_components, neg_mul_right, real_neg_zero
from complex.complex_seq import conj_seq, complex_limit, complex_converges, complex_limit_of_conj_seq
from complex.complex_exp import complex_exp, complex_exp_term, complex_exp_add, complex_exp_from_real, complex_exp_series_converges, complex_exp_i_mul_real_eq, complex_neg_one_new, complex_i_mul_real, complex_partial_zero, complex_partial_suc
from complex.complex_properties import mul_conj_modulus_squared
from complex.complex_pow import complex_conj_fn_pow
from complex.complex_trig import complex_new_eq_from_real_i
from complex.complex_log_deep import complex_exp_div
from complex.complex import mul_i
from complex.roots_of_unity import complex_exp_pow

// ---------------------------------------------------------------------------
// Local support.
// ---------------------------------------------------------------------------

/// Multiplication by i is the rotation (x, y) -> (-y, x) (restated).
theorem mul_i_complex(a: Complex) {
    Complex.i * a = Complex.new(-a.im, a.re)
} by {
    mul_i(a)
    Complex.i * a = Complex.new(-a.im, a.re)
}


// ---------------------------------------------------------------------------
// a. Conjugation commutes with the exponential.
// ---------------------------------------------------------------------------

/// Conjugation commutes with partial sums:
/// conj(partial(f, n)) = partial(conj(f), n).
theorem conj_seq_partial(f: Nat -> Complex, n: Nat) {
    conj_seq(partial(f), n) = partial(conj_seq(f), n)
} by {
    define p(k: Nat) -> Bool {
        conj_seq(partial(f), k) = partial(conj_seq(f), k)
    }
    complex_partial_zero(f)
    partial(f, Nat.0) = Complex.0
    conj_seq(partial(f), Nat.0) = Complex.0.conj
    conj_zero
    Complex.0.conj = Complex.0
    partial(conj_seq(f), Nat.0) = Complex.0
    conj_seq(partial(f), Nat.0) = partial(conj_seq(f), Nat.0)
    p(Nat.0)
    forall(k: Nat) {
        if p(k) {
            conj_seq(partial(f), k) = partial(conj_seq(f), k)
            complex_partial_suc(f, k)
            partial(f, k.suc) = partial(f, k) + f(k)
            conj_seq(partial(f), k.suc) = (partial(f, k) + f(k)).conj
            conj_add(partial(f, k), f(k))
            (partial(f, k) + f(k)).conj = partial(f, k).conj + f(k).conj
            conj_seq(partial(f), k) = partial(f, k).conj
            conj_seq(f, k) = f(k).conj
            conj_seq(partial(f), k.suc) = conj_seq(partial(f), k) + conj_seq(f, k)
            conj_seq(partial(f), k.suc) = partial(conj_seq(f), k) + conj_seq(f, k)
            complex_partial_suc(conj_seq(f), k)
            partial(conj_seq(f), k.suc) = partial(conj_seq(f), k) + conj_seq(f, k)
            conj_seq(partial(f), k.suc) = partial(conj_seq(f), k.suc)
            p(k.suc)
        }
    }
    p(Nat.0) and forall(k: Nat) { p(k) implies p(k.suc) }
    alt_induction(p)
    p(n)
}

/// Conjugation commutes with the terms of the complex exponential series:
/// conj(z^n / n!) = conj(z)^n / n!.
theorem complex_exp_term_conj(z: Complex, n: Nat) {
    complex_exp_term(z, n).conj = complex_exp_term(z.conj, n)
} by {
    complex_exp_term(z, n) = z.pow(n) / Complex.from_real(Real.from_rat(Rat.from_nat(n.factorial)))
    complex_exp_term(z, n).conj = (z.pow(n) / Complex.from_real(Real.from_rat(Rat.from_nat(n.factorial)))).conj
    conj_div(z.pow(n), Complex.from_real(Real.from_rat(Rat.from_nat(n.factorial))))
    (z.pow(n) / Complex.from_real(Real.from_rat(Rat.from_nat(n.factorial)))).conj =
        z.pow(n).conj / Complex.from_real(Real.from_rat(Rat.from_nat(n.factorial))).conj
    complex_conj_fn_pow(z, n)
    z.pow(n).conj = z.conj.pow(n)
    conj_from_real(Real.from_rat(Rat.from_nat(n.factorial)))
    Complex.from_real(Real.from_rat(Rat.from_nat(n.factorial))).conj =
        Complex.from_real(Real.from_rat(Rat.from_nat(n.factorial)))
    z.pow(n).conj / Complex.from_real(Real.from_rat(Rat.from_nat(n.factorial))).conj =
        z.conj.pow(n) / Complex.from_real(Real.from_rat(Rat.from_nat(n.factorial)))
    complex_exp_term(z.conj, n) =
        z.conj.pow(n) / Complex.from_real(Real.from_rat(Rat.from_nat(n.factorial)))
    complex_exp_term(z, n).conj = complex_exp_term(z.conj, n)
}

/// Conjugation commutes with the complex exponential:
/// conj(z.exp) = (conj(z)).exp.
theorem complex_exp_conj(z: Complex) {
    complex_exp(z).conj = complex_exp(z.conj)
} by {
    complex_exp(z) = complex_limit(partial(complex_exp_term(z)))
    complex_exp(z).conj = complex_limit(partial(complex_exp_term(z))).conj
    complex_exp_series_converges(z)
    complex_converges(partial(complex_exp_term(z)))
    complex_limit_of_conj_seq(partial(complex_exp_term(z)))
    complex_converges(partial(complex_exp_term(z))) implies complex_limit(conj_seq(partial(complex_exp_term(z)))) = complex_limit(partial(complex_exp_term(z))).conj
    complex_limit(conj_seq(partial(complex_exp_term(z)))) = complex_limit(partial(complex_exp_term(z))).conj
    complex_exp(z).conj = complex_limit(conj_seq(partial(complex_exp_term(z))))
    forall(n: Nat) {
        conj_seq_partial(complex_exp_term(z), n)
        conj_seq(partial(complex_exp_term(z)), n) = partial(conj_seq(complex_exp_term(z)), n)
    }
    conj_seq(partial(complex_exp_term(z))) = partial(conj_seq(complex_exp_term(z)))
    complex_limit(conj_seq(partial(complex_exp_term(z)))) =
        complex_limit(partial(conj_seq(complex_exp_term(z))))
    forall(n: Nat) {
        conj_seq(complex_exp_term(z), n) = complex_exp_term(z, n).conj
        complex_exp_term_conj(z, n)
        complex_exp_term(z, n).conj = complex_exp_term(z.conj, n)
        conj_seq(complex_exp_term(z), n) = complex_exp_term(z.conj, n)
    }
    conj_seq(complex_exp_term(z)) = complex_exp_term(z.conj)
    complex_limit(partial(conj_seq(complex_exp_term(z)))) =
        complex_limit(partial(complex_exp_term(z.conj)))
    complex_exp(z.conj) = complex_limit(partial(complex_exp_term(z.conj)))
    complex_exp(z).conj = complex_exp(z.conj)
}

// ---------------------------------------------------------------------------
// b. The modulus of the exponential.
// ---------------------------------------------------------------------------

/// The exponential of the sum of a number with its conjugate:
/// z.exp·(conj(z)).exp = (2·Re(z)).exp.
theorem complex_exp_mul_conj(z: Complex) {
    complex_exp(z) * complex_exp(z.conj) = Complex.from_real((z.re + z.re).exp)
} by {
    complex_exp_add(z, z.conj)
    complex_exp(z + z.conj) = complex_exp(z) * complex_exp(z.conj)
    z_add_conj(z)
    z + z.conj = Complex.from_real(z.re + z.re)
    complex_exp(z + z.conj) = complex_exp(Complex.from_real(z.re + z.re))
    complex_exp_from_real(z.re + z.re)
    complex_exp(Complex.from_real(z.re + z.re)) = Complex.from_real((z.re + z.re).exp)
    complex_exp(z) * complex_exp(z.conj) = Complex.from_real((z.re + z.re).exp)
}

/// The modulus of the exponential is the exponential of the real part:
/// |z.exp| = (Re(z)).exp.
theorem complex_exp_modulus(z: Complex) {
    complex_exp(z).modulus = (z.re).exp
} by {
    mul_conj_modulus_squared(complex_exp(z))
    complex_exp(z) * complex_exp(z).conj = Complex.from_real(complex_exp(z).modulus * complex_exp(z).modulus)
    complex_exp_conj(z)
    complex_exp(z).conj = complex_exp(z.conj)
    complex_exp(z) * complex_exp(z).conj = complex_exp(z) * complex_exp(z.conj)
    complex_exp_mul_conj(z)
    complex_exp(z) * complex_exp(z.conj) = Complex.from_real((z.re + z.re).exp)
    Complex.from_real(complex_exp(z).modulus * complex_exp(z).modulus) = Complex.from_real((z.re + z.re).exp)
    exp_add(z.re, z.re)
    (z.re + z.re).exp = (z.re).exp * (z.re).exp
    Complex.from_real((z.re + z.re).exp) = Complex.from_real((z.re).exp * (z.re).exp)
    real_mul_lifts((z.re).exp, (z.re).exp)
    Complex.from_real((z.re).exp * (z.re).exp) = Complex.from_real((z.re).exp) * Complex.from_real((z.re).exp)
    Complex.from_real(complex_exp(z).modulus * complex_exp(z).modulus) =
        Complex.from_real((z.re).exp) * Complex.from_real((z.re).exp)
    re_from_real(complex_exp(z).modulus * complex_exp(z).modulus)
    Complex.from_real(complex_exp(z).modulus * complex_exp(z).modulus).re =
        complex_exp(z).modulus * complex_exp(z).modulus
    re_mul(Complex.from_real((z.re).exp), Complex.from_real((z.re).exp))
    (Complex.from_real((z.re).exp) * Complex.from_real((z.re).exp)).re =
        Complex.from_real((z.re).exp).re * Complex.from_real((z.re).exp).re -
        Complex.from_real((z.re).exp).im * Complex.from_real((z.re).exp).im
    re_from_real((z.re).exp)
    Complex.from_real((z.re).exp).re = (z.re).exp
    im_from_real((z.re).exp)
    Complex.from_real((z.re).exp).im = Real.0
    Complex.from_real((z.re).exp).im * Complex.from_real((z.re).exp).im = Real.0
    (Complex.from_real((z.re).exp) * Complex.from_real((z.re).exp)).re = (z.re).exp * (z.re).exp
    Complex.from_real(complex_exp(z).modulus * complex_exp(z).modulus).re =
        (Complex.from_real((z.re).exp) * Complex.from_real((z.re).exp)).re
    complex_exp(z).modulus * complex_exp(z).modulus = (z.re).exp * (z.re).exp
    modulus_nonneg(complex_exp(z))
    complex_exp(z).modulus >= Real.0
    exp_pos(z.re)
    (z.re).exp > Real.0
    (z.re).exp >= Real.0
    nonneg_eq_of_squares_eq(complex_exp(z).modulus, (z.re).exp)
    complex_exp(z).modulus >= Real.0 and (z.re).exp >= Real.0 and complex_exp(z).modulus * complex_exp(z).modulus = (z.re).exp * (z.re).exp implies complex_exp(z).modulus = (z.re).exp
    complex_exp(z).modulus = (z.re).exp
}

/// The exponential of a purely imaginary number has unit modulus:
/// |(i·x).exp| = 1.
theorem complex_exp_unit_modulus(x: Real) {
    complex_exp(complex_i_mul_real(x)).modulus = Real.1
} by {
    complex_exp_modulus(complex_i_mul_real(x))
    complex_exp(complex_i_mul_real(x)).modulus = (complex_i_mul_real(x).re).exp
    complex_i_mul_real(x) = Complex.i * Complex.from_real(x)
    mul_i_complex(Complex.from_real(x))
    Complex.i * Complex.from_real(x) =
        Complex.new(-Complex.from_real(x).im, Complex.from_real(x).re)
    complex_i_mul_real(x) = Complex.new(-Complex.from_real(x).im, Complex.from_real(x).re)
    im_from_real(x)
    Complex.from_real(x).im = Real.0
    -Complex.from_real(x).im = -Real.0
    real_neg_zero
    -Real.0 = Real.0
    Complex.new(-Complex.from_real(x).im, Complex.from_real(x).re) = Complex.new(Real.0, Complex.from_real(x).re)
    complex_i_mul_real(x) = Complex.new(Real.0, Complex.from_real(x).re)
    complex_i_mul_real(x).re = Real.0
    (complex_i_mul_real(x).re).exp = (Real.0).exp
    exp_zero
    (Real.0).exp = Real.1
    complex_exp(complex_i_mul_real(x)).modulus = Real.1
}

// ---------------------------------------------------------------------------
// c. The Cartesian decomposition.
// ---------------------------------------------------------------------------

/// The exponential splits into real and imaginary parts:
/// z.exp = (Re(z)).exp·(i·Im(z)).exp.
theorem complex_exp_re_im_decomp(z: Complex) {
    complex_exp(z) = complex_exp(Complex.from_real(z.re)) * complex_exp(complex_i_mul_real(z.im))
} by {
    complex_eq_new_components(z)
    z = Complex.new(z.re, z.im)
    complex_new_eq_from_real_i(z.re, z.im)
    Complex.new(z.re, z.im) = Complex.from_real(z.re) + Complex.i * Complex.from_real(z.im)
    z = Complex.from_real(z.re) + Complex.i * Complex.from_real(z.im)
    complex_i_mul_real(z.im) = Complex.i * Complex.from_real(z.im)
    z = Complex.from_real(z.re) + complex_i_mul_real(z.im)
    complex_exp(z) = complex_exp(Complex.from_real(z.re) + complex_i_mul_real(z.im))
    complex_exp_add(Complex.from_real(z.re), complex_i_mul_real(z.im))
    complex_exp(Complex.from_real(z.re) + complex_i_mul_real(z.im)) =
        complex_exp(Complex.from_real(z.re)) * complex_exp(complex_i_mul_real(z.im))
    complex_exp(z) = complex_exp(Complex.from_real(z.re)) * complex_exp(complex_i_mul_real(z.im))
}

/// The Cartesian form of the exponential:
/// z.exp = (Re(z)).exp·((Im(z)).cos + i·(Im(z)).sin).
theorem complex_exp_cartesian(z: Complex) {
    complex_exp(z) =
        complex_exp(Complex.from_real(z.re)) *
            (Complex.from_real((z.im).cos) + Complex.i * Complex.from_real((z.im).sin))
} by {
    complex_exp_re_im_decomp(z)
    complex_exp(z) = complex_exp(Complex.from_real(z.re)) * complex_exp(complex_i_mul_real(z.im))
    complex_exp_i_mul_real_eq(z.im)
    complex_exp(complex_i_mul_real(z.im)) = Complex.new((z.im).cos, (z.im).sin)
    complex_new_eq_from_real_i((z.im).cos, (z.im).sin)
    Complex.new((z.im).cos, (z.im).sin) =
        Complex.from_real((z.im).cos) + Complex.i * Complex.from_real((z.im).sin)
    complex_exp(complex_i_mul_real(z.im)) =
        Complex.from_real((z.im).cos) + Complex.i * Complex.from_real((z.im).sin)
    complex_exp(z) =
        complex_exp(Complex.from_real(z.re)) *
            (Complex.from_real((z.im).cos) + Complex.i * Complex.from_real((z.im).sin))
}

// ---------------------------------------------------------------------------
// d. The remaining exponential laws.
// ---------------------------------------------------------------------------

/// The exponential of a difference is the quotient of exponentials:
/// (z - w).exp = z.exp / w.exp.
theorem complex_exp_sub_restated(z: Complex, w: Complex) {
    complex_exp(z - w) = complex_exp(z) / complex_exp(w)
} by {
    complex_exp_div(z, w)
    complex_exp(z - w) = complex_exp(z) * complex_exp(w).inverse
    complex_exp(z) * complex_exp(w).inverse = complex_exp(z) / complex_exp(w)
    complex_exp(z - w) = complex_exp(z) / complex_exp(w)
}

/// Natural multiples of the argument multiply the exponential:
/// (n·z).exp = z.exp^n.
theorem complex_exp_nat_mul(z: Complex, n: Nat) {
    complex_exp(Complex.from_real(from_nat[Real](n)) * z) = complex_exp(z).pow(n)
} by {
    complex_exp_pow(z, n)
    complex_exp(z).pow(n) = complex_exp(Complex.from_real(from_nat[Real](n)) * z)
    complex_exp(Complex.from_real(from_nat[Real](n)) * z) = complex_exp(z).pow(n)
}

/// The exponential flips sign under a shift by i·pi:
/// (z + i·pi).exp = -z.exp.
theorem complex_exp_add_pi(z: Complex) {
    complex_exp(z + complex_i_mul_real(pi)) = -complex_exp(z)
} by {
    complex_exp_add(z, complex_i_mul_real(pi))
    complex_exp(z + complex_i_mul_real(pi)) = complex_exp(z) * complex_exp(complex_i_mul_real(pi))
    complex_exp_i_mul_real_eq(pi)
    complex_exp(complex_i_mul_real(pi)) = Complex.new(pi.cos, pi.sin)
    cos_pi_neg_one
    pi.cos = -Real.1
    sin_pi_zero
    pi.sin = Real.0
    Complex.new(pi.cos, pi.sin) = Complex.new(-Real.1, Real.0)
    complex_neg_one_new
    Complex.new(-Real.1, Real.0) = -Complex.1
    complex_exp(complex_i_mul_real(pi)) = -Complex.1
    complex_exp(z) * complex_exp(complex_i_mul_real(pi)) = complex_exp(z) * (-Complex.1)
    neg_mul_right(complex_exp(z), Complex.1)
    complex_exp(z) * (-Complex.1) = -(complex_exp(z) * Complex.1)
    mul_one_right(complex_exp(z))
    complex_exp(z) * Complex.1 = complex_exp(z)
    complex_exp(z) * (-Complex.1) = -complex_exp(z)
    complex_exp(z + complex_i_mul_real(pi)) = -complex_exp(z)
}

