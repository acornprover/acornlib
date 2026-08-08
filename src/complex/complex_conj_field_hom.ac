from real import Real
from complex.complex import Complex, conj_from_real, is_real_imp_conj_eq, conj_eq_imp_is_real,
    conj_inverse, conj_div
from complex.complex_conj_hom import complex_conj_fn, complex_conj_fn_is_ring_hom,
    complex_conj_fn_compose_self
from algebra.field.field_hom import FieldHom, is_field_hom,
    identity_field_hom, identity_field_hom_hom,
    compose_field_hom, compose_field_hom_hom
from data.basic.functions import identity_fn, compose

/// Complex conjugation is a field homomorphism.
theorem complex_conj_fn_is_field_hom {
    is_field_hom(complex_conj_fn)
} by {
    complex_conj_fn_is_ring_hom
}

/// `FieldHom.new` produces a `Some` value for complex conjugation.
theorem complex_conj_fn_field_hom_some {
    exists(h: FieldHom[Complex, Complex]) {
        FieldHom.new(complex_conj_fn) = Option.some(h)
    }
} by {
    complex_conj_fn_is_field_hom
}

/// Complex conjugation as a field homomorphism of the complex numbers.
let complex_conj_field_hom: FieldHom[Complex, Complex] satisfy {
    FieldHom.new(complex_conj_fn) = Option.some(complex_conj_field_hom)
}

/// The underlying function of the complex conjugation field homomorphism is complex conjugation.
theorem complex_conj_field_hom_hom {
    complex_conj_field_hom.hom = complex_conj_fn
} by {
}

/// Extensionality specialized to complex field homomorphisms from equality of underlying functions.
theorem complex_field_hom_eq_of_hom_eq(f: FieldHom[Complex, Complex], g: FieldHom[Complex, Complex]) {
    f.hom = g.hom implies f = g
} by {
    if f.hom = g.hom {
        Option.some(f) = Option.some(g)
        f = g
    }
}

/// The field-hom formulation of conjugation commuting with multiplicative inverse.
theorem complex_conj_field_hom_inverse(a: Complex) {
    complex_conj_field_hom.hom(a.inverse) = complex_conj_field_hom.hom(a).inverse
} by {
    complex_conj_field_hom_hom
    complex_conj_fn(a.inverse) = a.inverse.conj
    complex_conj_fn(a) = a.conj
    conj_inverse(a)
}

/// The field-hom formulation of conjugation distributing over division.
theorem complex_conj_field_hom_div(a: Complex, b: Complex) {
    complex_conj_field_hom.hom(a / b) =
        complex_conj_field_hom.hom(a) / complex_conj_field_hom.hom(b)
} by {
    complex_conj_field_hom_hom
    complex_conj_fn(a / b) = (a / b).conj
    complex_conj_fn(a) = a.conj
    complex_conj_fn(b) = b.conj
    conj_div(a, b)
}

/// The composite of the complex conjugation field homomorphism with itself has
/// the identity function as its underlying map.
theorem complex_conj_field_hom_compose_self_hom {
    compose_field_hom(complex_conj_field_hom, complex_conj_field_hom).hom = identity_fn[Complex]
} by {
    compose_field_hom_hom(complex_conj_field_hom, complex_conj_field_hom)
    complex_conj_field_hom_hom
    complex_conj_fn_compose_self
}

/// The complex conjugation field homomorphism is involutive under composition.
theorem complex_conj_field_hom_involutive {
    compose_field_hom(complex_conj_field_hom, complex_conj_field_hom) =
        identity_field_hom[Complex]
} by {
    let lhs = compose_field_hom(complex_conj_field_hom, complex_conj_field_hom)
    let rhs = identity_field_hom[Complex]
    complex_conj_field_hom_compose_self_hom
    lhs.hom = identity_fn[Complex]
    identity_field_hom_hom[Complex]
    rhs.hom = identity_fn[Complex]
    lhs.hom = rhs.hom
    complex_field_hom_eq_of_hom_eq(lhs, rhs)
}

/// The complex conjugation field homomorphism fixes embedded real values.
theorem complex_conj_field_hom_from_real(a: Real) {
    complex_conj_field_hom.hom(Complex.from_real(a)) = Complex.from_real(a)
} by {
    complex_conj_field_hom_hom
    complex_conj_fn(Complex.from_real(a)) = Complex.from_real(a).conj
    conj_from_real(a)
}

/// Every real complex number is fixed by the complex conjugation field homomorphism.
theorem complex_conj_field_hom_is_real_imp_fixed(a: Complex) {
    a.is_real implies complex_conj_field_hom.hom(a) = a
} by {
    if a.is_real {
        complex_conj_field_hom_hom
        complex_conj_fn(a) = a.conj
        is_real_imp_conj_eq(a)
        complex_conj_field_hom.hom(a) = a
    }
}

/// Every fixed point of the complex conjugation field homomorphism is real.
theorem complex_conj_field_hom_fixed_imp_is_real(a: Complex) {
    complex_conj_field_hom.hom(a) = a implies a.is_real
} by {
    if complex_conj_field_hom.hom(a) = a {
        complex_conj_field_hom_hom
        conj_eq_imp_is_real(a)
        a.is_real
    }
}

/// Fixed points of the complex conjugation field homomorphism are exactly the real complex numbers.
theorem complex_conj_field_hom_fixed_iff_is_real(a: Complex) {
    (complex_conj_field_hom.hom(a) = a) = a.is_real
} by {
    if complex_conj_field_hom.hom(a) = a {
        complex_conj_field_hom_fixed_imp_is_real(a)
    }
    if a.is_real {
        complex_conj_field_hom_is_real_imp_fixed(a)
    }
}
