// ---------------------------------------------------------------------------
// Deepening of the algebraic structure of the complex numbers.
//
// This module extends src/complex/complex.ac with the elementary field laws
// that complex analysis uses constantly:
//
//   a. The additive group laws: negation of negation, negation of a sum,
//      subtraction identities (z - z = 0, (z + w) - w = z, z - (-w) = z + w,
//      -(z - w) = w - z, z - w = 0  iff  z = w) and the cancellation laws.
//   b. The ring laws for products: negating a product, squares of sums and
//      differences, the difference of squares, and the expansion
//      (a + i b)(a - i b) = a^2 + b^2.
//   c. The division laws: distributivity of division over addition, the
//      associativity of multiplication with division, negation of quotients,
//      the inverse of a negation, and 1 / z = z^(-1).
//   d. The powers of the imaginary unit: i is different from 0, 1 and -1,
//      the conjugate of i is -i, and the embedding of real powers lifts.
//   e. The decomposition of a complex number: z = Complex.new(z.re, z.im),
//      z + conj(z) = 2·Re(z) and z - conj(z) = 2·i·Im(z).
// ---------------------------------------------------------------------------

from real import Real, two
from nat import Nat, pow_zero, pow_one, one_pow, alt_induction
from algebra.add_group import inverse_inverse, inverse_add, left_cancel, right_cancel, inverse_left
from algebra.ring.ring import mul_sub_left, mul_neg_left, mul_neg_right, mul_neg_neg
from algebra.field.field import unique_inverse, inverse_one, mul_inverse_left
from complex.complex import Complex, eq_by_components, add_assoc, add_comm, re_add, im_add, re_sub, im_sub,
    re_neg, im_neg, re_conj, im_conj, re_i, im_i, re_from_real, im_from_real, add_neg_cancel,
    mul_comm, mul_assoc, distrib, left_distrib, mul_one_left, mul_one_right, mul_zero_left,
    add_zero_left, add_zero_right, i_squared_eq_neg_one, neg_eq_mul_neg_one, mul_inverse,
    zero_is_different_than_one, real_mul_lifts, real_add_lifts, from_real_one, from_real_neg,
    conj_conj, conj_neg, conj_sub, abs_squared_conj, is_real_neg, mul_left_cancel, mul_i, zero_inverse
from complex.complex_trig import complex_neg_zero, complex_from_real_two_nonzero
from complex.complex_exp import complex_pow_suc, complex_i_mul_real, real_pow_suc, complex_zero_pow_pos
from complex.complex_properties import pow_two_i, pow_two_neg_one

// ---------------------------------------------------------------------------
// Small local lemmas used above.
// ---------------------------------------------------------------------------

/// The negation of the real zero is zero.
theorem real_neg_zero {
    -Real.0 = Real.0
} by {
}

/// The real number two is one plus one.
theorem two_eq_real_one_plus_one {
    two = Real.1 + Real.1
} by {
}

// ---------------------------------------------------------------------------
// a. The additive group laws.
// ---------------------------------------------------------------------------

/// The negation of a negation is the original number: -(-z) = z.
theorem neg_neg_complex(a: Complex) {
    -(-a) = a
} by {
    inverse_inverse[Complex](a)
    -(-a) = a
}

/// The negation of a sum is the sum of the negations: -(z + w) = -z + -w.
theorem neg_add(a: Complex, b: Complex) {
    -(a + b) = -a + -b
} by {
    inverse_add[Complex](a, b)
    -(a + b) = -b + -a
    -b + -a = -a + -b
    -(a + b) = -a + -b
}

/// A complex number minus itself is zero: z - z = 0.
theorem sub_self_restated(a: Complex) {
    a - a = Complex.0
} by {
    a - a = a + -a
    add_neg_cancel(a)
    a + -a = Complex.0
    a - a = Complex.0
}

/// Adding and then subtracting the same number cancels: (z + w) - w = z.
theorem add_sub_cancel(a: Complex, b: Complex) {
    (a + b) - b = a
} by {
    (a + b) - b = (a + b) + -b
    add_assoc(a, b, -b)
    (a + b) + -b = a + (b + -b)
    add_neg_cancel(b)
    b + -b = Complex.0
    a + (b + -b) = a + Complex.0
    add_zero_right(a)
    a + Complex.0 = a
    (a + b) - b = a
}

/// Subtracting and then adding the same number cancels: (z - w) + w = z.
theorem sub_add_cancel(a: Complex, b: Complex) {
    (a - b) + b = a
} by {
    (a - b) + b = (a + -b) + b
    add_assoc(a, -b, b)
    (a + -b) + b = a + (-b + b)
    inverse_left[Complex](b)
    -b + b = Complex.0
    a + (-b + b) = a + Complex.0
    add_zero_right(a)
    a + Complex.0 = a
    (a - b) + b = a
}

/// Subtracting a negation is adding: z - (-w) = z + w.
theorem sub_neg_eq_add(a: Complex, b: Complex) {
    a - (-b) = a + b
} by {
    a - (-b) = a + -(-b)
    neg_neg_complex(b)
    -(-b) = b
    a + -(-b) = a + b
    a - (-b) = a + b
}

/// The negation of a difference is the reverse difference: -(z - w) = w - z.
theorem neg_sub(a: Complex, b: Complex) {
    -(a - b) = b - a
} by {
    a - b = a + -b
    -(a - b) = -(a + -b)
    neg_add(a, -b)
    -(a + -b) = -a + -(-b)
    neg_neg_complex(b)
    -(-b) = b
    -a + -(-b) = -a + b
    b - a = b + -a
    -a + b = b + -a
    -(a - b) = b - a
}

/// A difference is zero exactly when the two numbers agree: z - w = 0 iff z = w.
theorem sub_eq_zero_iff(a: Complex, b: Complex) {
    a - b = Complex.0 iff a = b
} by {
    if a - b = Complex.0 {
        a - b = a + -b
        a + -b = Complex.0
        (a + -b) + b = Complex.0 + b
        add_assoc(a, -b, b)
        (a + -b) + b = a + (-b + b)
        inverse_left[Complex](b)
        -b + b = Complex.0
        a + (-b + b) = a + Complex.0
        add_zero_right(a)
        a + Complex.0 = a
        add_zero_left(b)
        Complex.0 + b = b
        a = b
    }
    if a = b {
        a - b = a - a
        sub_self_restated(a)
        a - a = Complex.0
        a - b = Complex.0
    }
}

/// Adding the same number on the left is injective: z + w = z + v iff w = v.
theorem add_cancel_left(a: Complex, b: Complex, c: Complex) {
    a + b = a + c iff b = c
} by {
    if a + b = a + c {
        left_cancel[Complex](a, b, c)
        a + b = a + c implies b = c
        b = c
    }
    if b = c {
        a + b = a + c
    }
}

/// Adding the same number on the right is injective: z + w = v + w iff z = v.
theorem add_cancel_right(a: Complex, b: Complex, c: Complex) {
    a + c = b + c iff a = b
} by {
    if a + c = b + c {
        right_cancel[Complex](c, a, b)
        a + c = b + c implies a = b
        a = b
    }
    if a = b {
        a + c = b + c
    }
}

/// Telescoping: (a - b) + (b - c) = a - c.
theorem sub_add_telescope(a: Complex, b: Complex, c: Complex) {
    (a - b) + (b - c) = a - c
} by {
    (a - b) + (b - c) = (a + -b) + (b + -c)
    add_assoc(a, -b, b + -c)
    (a + -b) + (b + -c) = a + (-b + (b + -c))
    add_assoc(-b, b, -c)
    -b + (b + -c) = (-b + b) + -c
    inverse_left[Complex](b)
    -b + b = Complex.0
    (-b + b) + -c = Complex.0 + -c
    add_zero_left(-c)
    Complex.0 + -c = -c
    a + (-b + (b + -c)) = a + -c
    a - c = a + -c
    (a - b) + (b - c) = a - c
}

/// Doubling the difference form: (a + x) - (a - x) = 2·x.
theorem complex_add_sub_cancel_twice(a: Complex, x: Complex) {
    (a + x) - (a - x) = Complex.from_real(Real.1 + Real.1) * x
} by {
    (a + x) - (a - x) = (a + x) + -(a - x)
    neg_sub(a, x)
    -(a - x) = x - a
    (a + x) - (a - x) = (a + x) + (x - a)
    add_assoc(a, x, x - a)
    (a + x) + (x - a) = a + (x + (x - a))
    x + (x - a) = x + (x + -a)
    add_assoc(x, x, -a)
    x + (x + -a) = (x + x) + -a
    (x + x) + -a = (x + x) - a
    x + (x - a) = (x + x) - a
    a + (x + (x - a)) = a + ((x + x) - a)
    add_assoc(a, x + x, -a)
    a + ((x + x) - a) = (a + (x + x)) + -a
    add_comm(a, x + x)
    a + (x + x) = (x + x) + a
    (a + (x + x)) + -a = ((x + x) + a) + -a
    add_sub_cancel(x + x, a)
    ((x + x) + a) - a = x + x
    ((x + x) + a) + -a = x + x
    (a + x) - (a - x) = x + x
    real_add_lifts(Real.1, Real.1)
    Complex.from_real(Real.1 + Real.1) = Complex.from_real(Real.1) + Complex.from_real(Real.1)
    from_real_one
    Complex.from_real(Real.1) = Complex.1
    Complex.from_real(Real.1 + Real.1) = Complex.1 + Complex.1
    Complex.from_real(Real.1 + Real.1) * x = (Complex.1 + Complex.1) * x
    left_distrib(Complex.1, Complex.1, x)
    (Complex.1 + Complex.1) * x = Complex.1 * x + Complex.1 * x
    mul_one_left(x)
    Complex.1 * x = x
    (Complex.1 + Complex.1) * x = x + x
    Complex.from_real(Real.1 + Real.1) * x = x + x
    (a + x) - (a - x) = Complex.from_real(Real.1 + Real.1) * x
}

// ---------------------------------------------------------------------------
// b. The ring laws for products.
// ---------------------------------------------------------------------------

/// The product of two negations is the product of the originals: (-z)·(-w) = z·w.
theorem mul_neg_neg_restated(a: Complex, b: Complex) {
    (-a) * (-b) = a * b
} by {
    mul_neg_neg[Complex](a, b)
    (-a) * (-b) = a * b
}

/// A negation may be moved out of a product: -(z·w) = (-z)·w.
theorem neg_mul_left(a: Complex, b: Complex) {
    -(a * b) = (-a) * b
} by {
    mul_neg_left[Complex](a, b)
    (-a) * b = -(a * b)
    -(a * b) = (-a) * b
}

/// A negation may be moved out of a product on the right: -(z·w) = z·(-w).
theorem neg_mul_right(a: Complex, b: Complex) {
    -(a * b) = a * (-b)
} by {
    mul_neg_right[Complex](a, b)
    a * (-b) = -(a * b)
    -(a * b) = a * (-b)
}

/// The square agrees with the product: z² = z·z.
theorem pow_two_eq_mul_self(a: Complex) {
    a.pow(Nat.2) = a * a
} by {
    complex_pow_suc(a, Nat.1)
    a.pow(Nat.1.suc) = a * a.pow(Nat.1)
    Nat.1.suc = Nat.2
    a.pow(Nat.2) = a * a.pow(Nat.1)
    pow_one(a)
    a.pow(Nat.1) = a
    a.pow(Nat.2) = a * a
}

/// The square of a sum expands: (z + w)² = z² + 2·z·w + w².
theorem square_of_sum(a: Complex, b: Complex) {
    (a + b) * (a + b) = a * a + (a * b + a * b) + b * b
} by {
    distrib(a, b, a + b)
    (a + b) * (a + b) = a * (a + b) + b * (a + b)
    distrib(a, a, b)
    a * (a + b) = a * a + a * b
    distrib(b, a, b)
    b * (a + b) = b * a + b * b
    (a + b) * (a + b) = a * a + a * b + b * a + b * b
    mul_comm(b, a)
    b * a = a * b
    a * a + a * b + b * a + b * b = a * a + a * b + a * b + b * b
    (a + b) * (a + b) = a * a + (a * b + a * b) + b * b
}

/// The square of a difference expands: (z - w)² = z² - 2·z·w + w².
theorem square_of_diff(a: Complex, b: Complex) {
    (a - b) * (a - b) = a * a - (a * b + a * b) + b * b
} by {
    distrib(a, -b, a - b)
    (a - b) * (a - b) = a * (a - b) + (-b) * (a - b)
    mul_sub_left(a, a, b)
    a * (a - b) = a * a - a * b
    mul_sub_left(-b, a, b)
    (-b) * (a - b) = (-b) * a - (-b) * b
    mul_neg_left[Complex](b, a)
    (-b) * a = -(b * a)
    mul_neg_left[Complex](b, b)
    (-b) * b = -(b * b)
    (-b) * (a - b) = -(b * a) - (-(b * b))
    neg_neg_complex(b * b)
    -(-(b * b)) = b * b
    (-b) * (a - b) = -(b * a) + b * b
    (a - b) * (a - b) = a * a - a * b + (-(b * a) + b * b)
    mul_comm(b, a)
    b * a = a * b
    (a - b) * (a - b) = a * a - a * b + (-(a * b) + b * b)
    neg_add(a * b, a * b)
    -(a * b + a * b) = -(a * b) + -(a * b)
    add_assoc(a * a, -(a * b), -(a * b) + b * b)
    a * a - a * b + (-(a * b) + b * b) = a * a + (-(a * b) + (-(a * b) + b * b))
    add_assoc(-(a * b), -(a * b), b * b)
    (-(a * b) + -(a * b)) + b * b = -(a * b) + (-(a * b) + b * b)
    (-(a * b) + -(a * b)) + b * b = -(a * b + a * b) + b * b
    a * a + (-(a * b) + (-(a * b) + b * b)) = a * a + (-(a * b + a * b) + b * b)
    add_assoc(a * a, -(a * b + a * b), b * b)
    a * a + (-(a * b + a * b) + b * b) = (a * a + -(a * b + a * b)) + b * b
    a * a - (a * b + a * b) + b * b = (a * a + -(a * b + a * b)) + b * b
    a * a + (-(a * b) + (-(a * b) + b * b)) = a * a - (a * b + a * b) + b * b
    a * a - a * b + (-(a * b) + b * b) = a * a - (a * b + a * b) + b * b
    (a - b) * (a - b) = a * a - (a * b + a * b) + b * b
}

/// The difference of squares factors: (z + w)·(z - w) = z² - w².
theorem diff_of_squares(a: Complex, b: Complex) {
    (a + b) * (a - b) = a * a - b * b
} by {
    distrib(a, b, a - b)
    (a + b) * (a - b) = a * (a - b) + b * (a - b)
    mul_sub_left(a, a, b)
    a * (a - b) = a * a - a * b
    mul_sub_left(b, a, b)
    b * (a - b) = b * a - b * b
    (a + b) * (a - b) = a * a - a * b + (b * a - b * b)
    mul_comm(b, a)
    b * a = a * b
    (a + b) * (a - b) = a * a - a * b + (a * b - b * b)
    sub_add_telescope(a * a, a * b, b * b)
    (a * a - a * b) + (a * b - b * b) = a * a - b * b
    (a + b) * (a - b) = a * a - b * b
}

/// The expansion (a + i·b)·(a - i·b) = a² + b².
theorem mul_conj_expansion(a: Complex, b: Complex) {
    (a + Complex.i * b) * (a - Complex.i * b) = a * a + b * b
} by {
    distrib(a, Complex.i * b, a - Complex.i * b)
    (a + Complex.i * b) * (a - Complex.i * b) =
        a * (a - Complex.i * b) + (Complex.i * b) * (a - Complex.i * b)
    mul_sub_left(a, a, Complex.i * b)
    a * (a - Complex.i * b) = a * a - a * (Complex.i * b)
    mul_sub_left(Complex.i * b, a, Complex.i * b)
    (Complex.i * b) * (a - Complex.i * b) =
        (Complex.i * b) * a - (Complex.i * b) * (Complex.i * b)
    (a + Complex.i * b) * (a - Complex.i * b) =
        (a * a - a * (Complex.i * b)) + ((Complex.i * b) * a - (Complex.i * b) * (Complex.i * b))
    mul_comm(a, Complex.i * b)
    a * (Complex.i * b) = (Complex.i * b) * a
    (a * a - a * (Complex.i * b)) + ((Complex.i * b) * a - (Complex.i * b) * (Complex.i * b)) =
        (a * a - (Complex.i * b) * a) + ((Complex.i * b) * a - (Complex.i * b) * (Complex.i * b))
    sub_add_telescope(a * a, (Complex.i * b) * a, (Complex.i * b) * (Complex.i * b))
    (a * a - (Complex.i * b) * a) + ((Complex.i * b) * a - (Complex.i * b) * (Complex.i * b)) =
        a * a - (Complex.i * b) * (Complex.i * b)
    (a + Complex.i * b) * (a - Complex.i * b) = a * a - (Complex.i * b) * (Complex.i * b)
    mul_comm(Complex.i, b)
    Complex.i * b = b * Complex.i
    (Complex.i * b) * (Complex.i * b) = (b * Complex.i) * (Complex.i * b)
    mul_assoc(b, Complex.i, Complex.i * b)
    (b * Complex.i) * (Complex.i * b) = b * (Complex.i * (Complex.i * b))
    mul_assoc(Complex.i, Complex.i, b)
    Complex.i * (Complex.i * b) = (Complex.i * Complex.i) * b
    i_squared_eq_neg_one
    Complex.i * Complex.i = -Complex.1
    (Complex.i * Complex.i) * b = (-Complex.1) * b
    neg_eq_mul_neg_one(b)
    -b = (-Complex.1) * b
    (-Complex.1) * b = -b
    b * (Complex.i * (Complex.i * b)) = b * (-b)
    mul_neg_right(b, b)
    b * (-b) = -(b * b)
    (Complex.i * b) * (Complex.i * b) = -(b * b)
    a * a - (-(b * b)) = a * a + -(-(b * b))
    neg_neg_complex(b * b)
    -(-(b * b)) = b * b
    a * a + -(-(b * b)) = a * a + b * b
    a * a - (-(b * b)) = a * a + b * b
    (a + Complex.i * b) * (a - Complex.i * b) = a * a + b * b
}

// ---------------------------------------------------------------------------
// c. The division laws.
// ---------------------------------------------------------------------------

/// Division distributes over addition: (z + w) / c = z / c + w / c.
theorem div_distrib_right(a: Complex, b: Complex, c: Complex) {
    (a + b) / c = a / c + b / c
} by {
    (a + b) / c = (a + b) * c.inverse
    distrib(a, b, c.inverse)
    (a + b) * c.inverse = a * c.inverse + b * c.inverse
    a * c.inverse = a / c
    b * c.inverse = b / c
    (a + b) / c = a / c + b / c
}

/// The negation of a quotient is the quotient of the negation: -(z / w) = (-z) / w.
theorem neg_div_num(a: Complex, b: Complex) {
    -(a / b) = (-a) / b
} by {
    a / b = a * b.inverse
    -(a / b) = -(a * b.inverse)
    mul_neg_left[Complex](a, b.inverse)
    (-a) * b.inverse = -(a * b.inverse)
    (-a) / b = (-a) * b.inverse
    -(a / b) = (-a) / b
}

/// The inverse of a negation is the negation of the inverse: (-z)^(-1) = -z^(-1).
theorem inverse_neg(a: Complex) {
    (-a).inverse = -a.inverse
} by {
    if a = Complex.0 {
        (-a).inverse = (-Complex.0).inverse
        complex_neg_zero
        -Complex.0 = Complex.0
        (-a).inverse = Complex.0.inverse
        zero_inverse
        Complex.0.inverse = Complex.0
        (-a).inverse = Complex.0
        -a.inverse = -Complex.0.inverse
        -a.inverse = -Complex.0
        complex_neg_zero
        -Complex.0 = Complex.0
        -a.inverse = Complex.0
        (-a).inverse = -a.inverse
    } else {
        a != Complex.0
        mul_inverse(a)
        a * a.inverse = Complex.1
        mul_neg_neg[Complex](a, a.inverse)
        (-a) * (-a.inverse) = a * a.inverse
        (-a) * (-a.inverse) = Complex.1
        unique_inverse[Complex](-a, -a.inverse)
        (-a) * (-a.inverse) = Complex.1 implies -a.inverse = (-a).inverse
        -a.inverse = (-a).inverse
        (-a).inverse = -a.inverse
    }
}

/// Division by a negation negates the quotient: z / (-w) = -(z / w).
theorem neg_div(a: Complex, b: Complex) {
    a / (-b) = -(a / b)
} by {
    a / (-b) = a * (-b).inverse
    inverse_neg(b)
    (-b).inverse = -b.inverse
    a * (-b).inverse = a * (-b.inverse)
    mul_neg_right[Complex](a, b.inverse)
    a * (-b.inverse) = -(a * b.inverse)
    a * b.inverse = a / b
    a / (-b) = -(a / b)
}

/// The inverse of the inverse is the original number: (z^(-1))^(-1) = z.
theorem inverse_inverse_restated(a: Complex) {
    a.inverse.inverse = a
} by {
    if a = Complex.0 {
        a.inverse = Complex.0
        a.inverse.inverse = Complex.0.inverse
        zero_inverse
        Complex.0.inverse = Complex.0
        a.inverse.inverse = Complex.0
        a.inverse.inverse = a
    } else {
        a != Complex.0
        mul_inverse_left[Complex](a)
        a.inverse * a = Complex.1
        unique_inverse[Complex](a.inverse, a)
        a.inverse * a = Complex.1 implies a = a.inverse.inverse
        a = a.inverse.inverse
        a.inverse.inverse = a
    }
}

/// The inverse of one is one.
theorem inverse_one_restated {
    Complex.1.inverse = Complex.1
} by {
    inverse_one[Complex]
    Complex.1.inverse = Complex.1
}

/// Dividing by a number multiplies by its inverse: 1 / z = z^(-1).
theorem one_div_inverse(a: Complex) {
    Complex.1 / a = a.inverse
} by {
    Complex.1 / a = Complex.1 * a.inverse
    mul_one_left(a.inverse)
    Complex.1 * a.inverse = a.inverse
    Complex.1 / a = a.inverse
}

/// Multiplication associates with division: (z·w) / c = z·(w / c).
theorem mul_div_assoc(a: Complex, b: Complex, c: Complex) {
    (a * b) / c = a * (b / c)
} by {
    (a * b) / c = (a * b) * c.inverse
    mul_assoc(a, b, c.inverse)
    (a * b) * c.inverse = a * (b * c.inverse)
    b * c.inverse = b / c
    a * (b * c.inverse) = a * (b / c)
    (a * b) / c = a * (b / c)
}

// ---------------------------------------------------------------------------
// d. The imaginary unit and the real embedding.
// ---------------------------------------------------------------------------

/// One is not negative one.
theorem one_neq_neg_one_complex {
    Complex.1 != -Complex.1
} by {
    if Complex.1 = -Complex.1 {
        Complex.1 + Complex.1 = -Complex.1 + Complex.1
        -Complex.1 + Complex.1 = Complex.0
        Complex.1 + Complex.1 = Complex.0
        real_add_lifts(Real.1, Real.1)
        Complex.from_real(Real.1 + Real.1) = Complex.from_real(Real.1) + Complex.from_real(Real.1)
        from_real_one
        Complex.from_real(Real.1) = Complex.1
        Complex.from_real(Real.1 + Real.1) = Complex.1 + Complex.1
        two_eq_real_one_plus_one
        two = Real.1 + Real.1
        Complex.from_real(two) = Complex.from_real(Real.1 + Real.1)
        Complex.from_real(two) = Complex.1 + Complex.1
        Complex.from_real(two) = Complex.0
        complex_from_real_two_nonzero
        Complex.from_real(two) != Complex.0
        false
    }
    Complex.1 != -Complex.1
}

/// The imaginary unit is not zero.
theorem i_neq_zero {
    Complex.i != Complex.0
} by {
    if Complex.i = Complex.0 {
        Complex.i.pow(Nat.2) = Complex.0.pow(Nat.2)
        pow_two_i
        Complex.i.pow(Nat.2) = -Complex.1
        pow_two_eq_mul_self(Complex.0)
        Complex.0.pow(Nat.2) = Complex.0 * Complex.0
        mul_zero_left(Complex.0)
        Complex.0 * Complex.0 = Complex.0
        Complex.0.pow(Nat.2) = Complex.0
        -Complex.1 = Complex.0
        -Complex.1 + Complex.1 = Complex.0 + Complex.1
        -Complex.1 + Complex.1 = Complex.0
        add_zero_left(Complex.1)
        Complex.0 + Complex.1 = Complex.1
        Complex.0 = Complex.1
        zero_is_different_than_one
        Complex.0 != Complex.1
        false
    }
    Complex.i != Complex.0
}

/// The imaginary unit is not one.
theorem i_neq_one {
    Complex.i != Complex.1
} by {
    if Complex.i = Complex.1 {
        Complex.i.pow(Nat.2) = Complex.1.pow(Nat.2)
        pow_two_i
        Complex.i.pow(Nat.2) = -Complex.1
        one_pow[Complex](Nat.2)
        Complex.1.pow(Nat.2) = Complex.1
        -Complex.1 = Complex.1
        one_neq_neg_one_complex
        Complex.1 != -Complex.1
        false
    }
    Complex.i != Complex.1
}

/// The imaginary unit is not negative one.
theorem i_neq_neg_one {
    Complex.i != -Complex.1
} by {
    if Complex.i = -Complex.1 {
        Complex.i.pow(Nat.2) = (-Complex.1).pow(Nat.2)
        pow_two_i
        Complex.i.pow(Nat.2) = -Complex.1
        pow_two_neg_one
        (-Complex.1).pow(Nat.2) = Complex.1
        -Complex.1 = Complex.1
        one_neq_neg_one_complex
        Complex.1 != -Complex.1
        false
    }
    Complex.i != -Complex.1
}

/// The conjugate of the imaginary unit is -i.
theorem conj_i {
    Complex.i.conj = -Complex.i
} by {
    re_conj(Complex.i)
    Complex.i.conj.re = Complex.i.re
    re_i
    Complex.i.re = Real.0
    Complex.i.conj.re = Real.0
    re_neg(Complex.i)
    (-Complex.i).re = -Complex.i.re
    real_neg_zero
    -Real.0 = Real.0
    (-Complex.i).re = Real.0
    Complex.i.conj.re = (-Complex.i).re
    im_conj(Complex.i)
    Complex.i.conj.im = -Complex.i.im
    im_i
    Complex.i.im = Real.1
    Complex.i.conj.im = -Real.1
    im_neg(Complex.i)
    (-Complex.i).im = -Complex.i.im
    (-Complex.i).im = -Real.1
    Complex.i.conj.im = (-Complex.i).im
    eq_by_components(Complex.i.conj, -Complex.i)
    Complex.i.conj = -Complex.i
}

/// The embedding of a real power is the power of the embedding:
/// from_real(x)^n = from_real(x^n).
theorem from_real_pow(x: Real, n: Nat) {
    Complex.from_real(x).pow(n) = Complex.from_real(x.pow(n))
} by {
    define p(k: Nat) -> Bool {
        Complex.from_real(x).pow(k) = Complex.from_real(x.pow(k))
    }
    pow_zero(Complex.from_real(x))
    Complex.from_real(x).pow(Nat.0) = Complex.1
    pow_zero(x)
    x.pow(Nat.0) = Real.1
    from_real_one
    Complex.from_real(Real.1) = Complex.1
    Complex.from_real(x.pow(Nat.0)) = Complex.from_real(Real.1)
    Complex.from_real(x.pow(Nat.0)) = Complex.1
    Complex.from_real(x).pow(Nat.0) = Complex.from_real(x.pow(Nat.0))
    p(Nat.0)
    forall(k: Nat) {
        if p(k) {
            Complex.from_real(x).pow(k) = Complex.from_real(x.pow(k))
            complex_pow_suc(Complex.from_real(x), k)
            Complex.from_real(x).pow(k.suc) = Complex.from_real(x) * Complex.from_real(x).pow(k)
            Complex.from_real(x).pow(k.suc) = Complex.from_real(x) * Complex.from_real(x.pow(k))
            real_pow_suc(x, k)
            x.pow(k.suc) = x * x.pow(k)
            real_mul_lifts(x, x.pow(k))
            Complex.from_real(x * x.pow(k)) = Complex.from_real(x) * Complex.from_real(x.pow(k))
            Complex.from_real(x.pow(k.suc)) = Complex.from_real(x) * Complex.from_real(x.pow(k))
            Complex.from_real(x).pow(k.suc) = Complex.from_real(x.pow(k.suc))
            p(k.suc)
        }
    }
    p(Nat.0) and forall(k: Nat) { p(k) implies p(k.suc) }
    alt_induction(p)
    p(n)
}

/// The square of an embedded real is the embedding of the square:
/// from_real(x)² = from_real(x²).
theorem from_real_sq(x: Real) {
    Complex.from_real(x) * Complex.from_real(x) = Complex.from_real(x * x)
} by {
    real_mul_lifts(x, x)
    Complex.from_real(x * x) = Complex.from_real(x) * Complex.from_real(x)
    Complex.from_real(x) * Complex.from_real(x) = Complex.from_real(x * x)
}

// ---------------------------------------------------------------------------
// e. The decomposition of a complex number.
// ---------------------------------------------------------------------------

/// Every complex number equals the pair of its real and imaginary parts.
theorem complex_eq_new_components(z: Complex) {
    z = Complex.new(z.re, z.im)
} by {
    Complex.new(z.re, z.im).re = z.re
    Complex.new(z.re, z.im).im = z.im
    eq_by_components(z, Complex.new(z.re, z.im))
    z = Complex.new(z.re, z.im)
}

/// The sum of a complex number and its conjugate is twice the real part:
/// z + conj(z) = 2·Re(z).
theorem z_add_conj(z: Complex) {
    z + z.conj = Complex.from_real(z.re + z.re)
} by {
    re_add(z, z.conj)
    (z + z.conj).re = z.re + z.conj.re
    re_conj(z)
    z.conj.re = z.re
    (z + z.conj).re = z.re + z.re
    im_add(z, z.conj)
    (z + z.conj).im = z.im + z.conj.im
    im_conj(z)
    z.conj.im = -z.im
    (z + z.conj).im = z.im + -z.im
    inverse_left[Real](z.im)
    -z.im + z.im = Real.0
    z.im + -z.im = -z.im + z.im
    z.im + -z.im = Real.0
    (z + z.conj).im = Real.0
    re_from_real(z.re + z.re)
    Complex.from_real(z.re + z.re).re = z.re + z.re
    im_from_real(z.re + z.re)
    Complex.from_real(z.re + z.re).im = Real.0
    eq_by_components(z + z.conj, Complex.from_real(z.re + z.re))
    z + z.conj = Complex.from_real(z.re + z.re)
}

/// The difference of a complex number and its conjugate is twice the imaginary
/// part times i: z - conj(z) = 2·i·Im(z).
theorem z_sub_conj(z: Complex) {
    z - z.conj = Complex.i * Complex.from_real(z.im + z.im)
} by {
    re_sub(z, z.conj)
    (z - z.conj).re = z.re - z.conj.re
    re_conj(z)
    z.conj.re = z.re
    (z - z.conj).re = z.re - z.re
    inverse_left[Real](z.re)
    -z.re + z.re = Real.0
    z.re + -z.re = -z.re + z.re
    z.re + -z.re = Real.0
    z.re - z.re = z.re + -z.re
    (z - z.conj).re = Real.0
    im_sub(z, z.conj)
    (z - z.conj).im = z.im - z.conj.im
    im_conj(z)
    z.conj.im = -z.im
    (z - z.conj).im = z.im - (-z.im)
    inverse_inverse[Real](z.im)
    -(-z.im) = z.im
    z.im - (-z.im) = z.im + z.im
    (z - z.conj).im = z.im + z.im
    complex_i_mul_real(z.im + z.im) = Complex.i * Complex.from_real(z.im + z.im)
    mul_i(Complex.from_real(z.im + z.im))
    Complex.i * Complex.from_real(z.im + z.im) =
        Complex.new(-Complex.from_real(z.im + z.im).im, Complex.from_real(z.im + z.im).re)
    im_from_real(z.im + z.im)
    Complex.from_real(z.im + z.im).im = Real.0
    -Complex.from_real(z.im + z.im).im = -Real.0
    real_neg_zero
    -Real.0 = Real.0
    re_from_real(z.im + z.im)
    Complex.from_real(z.im + z.im).re = z.im + z.im
    Complex.i * Complex.from_real(z.im + z.im) = Complex.new(Real.0, z.im + z.im)
    Complex.new(Real.0, z.im + z.im).re = Real.0
    Complex.new(Real.0, z.im + z.im).im = z.im + z.im
    (z - z.conj).re = (Complex.i * Complex.from_real(z.im + z.im)).re
    (z - z.conj).im = (Complex.i * Complex.from_real(z.im + z.im)).im
    eq_by_components(z - z.conj, Complex.i * Complex.from_real(z.im + z.im))
    z - z.conj = Complex.i * Complex.from_real(z.im + z.im)
}

