from data.basic.set import Set
from real import Real, lte_trans, lte_antisymm, lte_lt_trans, lt_lte_trans, lte_add_right,
    abs_gte_zero
from complex.complex import Complex
from algebra.add_comm_monoid_rearrange import add_swap_inner
from complex.complex_abs import modulus_nonneg, modulus_neg, modulus_triangle, modulus_of_zero,
    modulus_eq_zero

/// True if `z` lies within `radius` of `center`.
///
/// The membership condition of a closed disk, named so that the region below is a set built
/// from a one-variable predicate rather than from an inline lambda.
define in_closed_disk(center: Complex, radius: Real, z: Complex) -> Bool {
    (z - center).modulus <= radius
}

/// The closed disk of a given centre and radius.
///
/// Regions of the complex plane are `Set[Complex]`, so that containment and equality of regions
/// are the containment and equality the set API already supplies.
define closed_disk(center: Complex, radius: Real) -> Set[Complex] {
    Set[Complex].new(in_closed_disk(center, radius))
}

/// Membership in a closed disk is being within the radius of the centre.
theorem closed_disk_contains_eq(center: Complex, radius: Real, z: Complex) {
    closed_disk(center, radius).contains(z) = ((z - center).modulus <= radius)
} by {
    (closed_disk(center, radius).contains = in_closed_disk(center, radius))
    closed_disk(center, radius).contains(z) = in_closed_disk(center, radius, z)
    (in_closed_disk(center, radius, z) = ((z - center).modulus <= radius))
}

/// A point within the radius lies in the disk.
theorem closed_disk_contains(center: Complex, radius: Real, z: Complex) {
    (z - center).modulus <= radius implies closed_disk(center, radius).contains(z)
} by {
    if (z - center).modulus <= radius {
        closed_disk_contains_eq(center, radius, z)
        closed_disk(center, radius).contains(z)
    }
}

/// A member of the disk is within the radius.
theorem closed_disk_member_bound(center: Complex, radius: Real, z: Complex) {
    closed_disk(center, radius).contains(z) implies (z - center).modulus <= radius
} by {
    if closed_disk(center, radius).contains(z) {
        closed_disk_contains_eq(center, radius, z)
        (z - center).modulus <= radius
    }
}

/// The centre lies in every disk of nonnegative radius.
theorem closed_disk_contains_center(center: Complex, radius: Real) {
    Real.0 <= radius implies closed_disk(center, radius).contains(center)
} by {
    if Real.0 <= radius {
        (center - center = Complex.0)
        modulus_of_zero
        Complex.0.modulus = Real.0
        (center - center).modulus = Real.0
        (center - center).modulus <= radius
        closed_disk_contains(center, radius, center)
        closed_disk(center, radius).contains(center)
    }
}

/// Enlarging the radius enlarges the disk.
theorem closed_disk_radius_mono(center: Complex, r: Real, s: Real) {
    r <= s implies closed_disk(center, r).subset(closed_disk(center, s))
} by {
    if r <= s {
        forall(z: Complex) {
            if closed_disk(center, r).contains(z) {
                closed_disk_member_bound(center, r, z)
                (z - center).modulus <= r
                lte_trans((z - center).modulus, r, s)
                (z - center).modulus <= s
                closed_disk_contains(center, s, z)
                closed_disk(center, s).contains(z)
            }
            (closed_disk(center, r).contains(z) implies closed_disk(center, s).contains(z))
        }
        (closed_disk(center, r).subset(closed_disk(center, s)) = forall(w: Complex) {
            closed_disk(center, r).contains(w) implies closed_disk(center, s).contains(w)
        })
        closed_disk(center, r).subset(closed_disk(center, s))
    }
}

/// A disk of zero radius is exactly its centre.
theorem closed_disk_zero_radius(center: Complex, z: Complex) {
    closed_disk(center, Real.0).contains(z) = (z = center)
} by {
    if closed_disk(center, Real.0).contains(z) {
        closed_disk_member_bound(center, Real.0, z)
        (z - center).modulus <= Real.0
        modulus_nonneg(z - center)
        (z - center).modulus >= Real.0
        lte_antisymm((z - center).modulus, Real.0)
        (z - center).modulus = Real.0
        modulus_eq_zero(z - center)
        z - center = Complex.0
        ((z - center) + center = Complex.0 + center)
        (z - center = z + -center)
        ((z + -center) + center = z + (-center + center))
        (-center + center = Complex.0)
        (z + Complex.0 = z)
        (Complex.0 + center = center)
        z = center
    }
    if z = center {
        Real.0 <= Real.0
        closed_disk_contains_center(center, Real.0)
        closed_disk(center, Real.0).contains(center)
        closed_disk(center, Real.0).contains(z)
    }
    (closed_disk(center, Real.0).contains(z) implies z = center)
    ((z = center) implies closed_disk(center, Real.0).contains(z))
    closed_disk(center, Real.0).contains(z) = (z = center)
}

/// A disk sits inside a bigger disk about any centre within reach.
///
/// If the centres are within `d` of each other, everything within `r` of the first is within
/// `d + r` of the second. This is the containment a perturbation argument uses.
theorem closed_disk_shift(c1: Complex, c2: Complex, r: Real, d: Real) {
    (c1 - c2).modulus <= d
        implies closed_disk(c1, r).subset(closed_disk(c2, d + r))
} by {
    if (c1 - c2).modulus <= d {
        forall(z: Complex) {
            if closed_disk(c1, r).contains(z) {
                closed_disk_member_bound(c1, r, z)
                (z - c1).modulus <= r
                (c1 - c2 = c1 + -c2)
                (z - c1 = z + -c1)
                add_swap_inner[Complex](c1, -c2, z, -c1)
                ((c1 + -c2) + (z + -c1) = (c1 + z) + (-c2 + -c1))
                add_swap_inner[Complex](c1, z, -c2, -c1)
                ((c1 + z) + (-c2 + -c1) = (c1 + -c2) + (z + -c1))
                add_swap_inner[Complex](c1, -c1, z, -c2)
                ((c1 + -c1) + (z + -c2) = (c1 + z) + (-c1 + -c2))
                (-c1 + -c2 = -c2 + -c1)
                ((c1 + -c1) + (z + -c2) = (c1 + z) + (-c2 + -c1))
                (c1 + -c1 = Complex.0)
                (Complex.0 + (z + -c2) = z + -c2)
                ((c1 + z) + (-c2 + -c1) = z + -c2)
                (z - c2 = z + -c2)
                (z - c2 = (c1 - c2) + (z - c1))
                modulus_triangle(c1 - c2, z - c1)
                ((c1 - c2) + (z - c1)).modulus <= (c1 - c2).modulus + (z - c1).modulus
                (z - c2).modulus <= (c1 - c2).modulus + (z - c1).modulus
                ((c1 - c2).modulus + (z - c1).modulus <= d + r)
                lte_trans((z - c2).modulus, (c1 - c2).modulus + (z - c1).modulus, d + r)
                (z - c2).modulus <= d + r
                closed_disk_contains(c2, d + r, z)
                closed_disk(c2, d + r).contains(z)
            }
            (closed_disk(c1, r).contains(z) implies closed_disk(c2, d + r).contains(z))
        }
        (closed_disk(c1, r).subset(closed_disk(c2, d + r)) = forall(w: Complex) {
            closed_disk(c1, r).contains(w) implies closed_disk(c2, d + r).contains(w)
        })
        closed_disk(c1, r).subset(closed_disk(c2, d + r))
    }
}

/// The closed unit disk about the origin.
///
/// The region an eigenvalue bound of one places a spectrum in.
let unit_disk: Set[Complex] = closed_disk(Complex.0, Real.1)

/// Membership in the unit disk is having modulus at most one.
theorem unit_disk_contains_eq(z: Complex) {
    unit_disk.contains(z) = (z.modulus <= Real.1)
} by {
    (unit_disk = closed_disk(Complex.0, Real.1))
    closed_disk_contains_eq(Complex.0, Real.1, z)
    (closed_disk(Complex.0, Real.1).contains(z) = ((z - Complex.0).modulus <= Real.1))
    (z - Complex.0 = z)
    unit_disk.contains(z) = (z.modulus <= Real.1)
}

/// A point of modulus at most one lies in the unit disk.
theorem unit_disk_contains(z: Complex) {
    z.modulus <= Real.1 implies unit_disk.contains(z)
} by {
    if z.modulus <= Real.1 {
        unit_disk_contains_eq(z)
        unit_disk.contains(z)
    }
}

/// True if `z` lies strictly within `radius` of `center`.
define in_open_disk(center: Complex, radius: Real, z: Complex) -> Bool {
    (z - center).modulus < radius
}

/// The open disk of a given centre and radius.
define open_disk(center: Complex, radius: Real) -> Set[Complex] {
    Set[Complex].new(in_open_disk(center, radius))
}

/// Membership in an open disk is being strictly within the radius of the centre.
theorem open_disk_contains_eq(center: Complex, radius: Real, z: Complex) {
    open_disk(center, radius).contains(z) = ((z - center).modulus < radius)
} by {
    (open_disk(center, radius).contains = in_open_disk(center, radius))
    open_disk(center, radius).contains(z) = in_open_disk(center, radius, z)
    (in_open_disk(center, radius, z) = ((z - center).modulus < radius))
}

/// An open disk sits inside the closed disk of the same radius.
theorem open_disk_subset_closed(center: Complex, radius: Real) {
    open_disk(center, radius).subset(closed_disk(center, radius))
} by {
    forall(z: Complex) {
        if open_disk(center, radius).contains(z) {
            open_disk_contains_eq(center, radius, z)
            (z - center).modulus < radius
            (z - center).modulus <= radius
            closed_disk_contains(center, radius, z)
            closed_disk(center, radius).contains(z)
        }
        (open_disk(center, radius).contains(z)
            implies closed_disk(center, radius).contains(z))
    }
    (open_disk(center, radius).subset(closed_disk(center, radius)) = forall(w: Complex) {
        open_disk(center, radius).contains(w) implies closed_disk(center, radius).contains(w)
    })
    open_disk(center, radius).subset(closed_disk(center, radius))
}

/// A closed disk sits inside every strictly larger open disk.
///
/// With the previous containment this is the sense in which the two families interleave, which
/// is what makes either of them enough to describe a region up to arbitrary precision.
theorem closed_disk_subset_open(center: Complex, r: Real, s: Real) {
    r < s implies closed_disk(center, r).subset(open_disk(center, s))
} by {
    if r < s {
        forall(z: Complex) {
            if closed_disk(center, r).contains(z) {
                closed_disk_member_bound(center, r, z)
                (z - center).modulus <= r
                lte_lt_trans((z - center).modulus, r, s)
                (z - center).modulus < s
                open_disk_contains_eq(center, s, z)
                open_disk(center, s).contains(z)
            }
            (closed_disk(center, r).contains(z) implies open_disk(center, s).contains(z))
        }
        (closed_disk(center, r).subset(open_disk(center, s)) = forall(w: Complex) {
            closed_disk(center, r).contains(w) implies open_disk(center, s).contains(w)
        })
        closed_disk(center, r).subset(open_disk(center, s))
    }
}

/// An open disk of nonpositive radius is empty.
theorem open_disk_nonpositive_empty(center: Complex, radius: Real, z: Complex) {
    radius <= Real.0 implies not open_disk(center, radius).contains(z)
} by {
    if radius <= Real.0 {
        if open_disk(center, radius).contains(z) {
            open_disk_contains_eq(center, radius, z)
            (z - center).modulus < radius
            lt_lte_trans((z - center).modulus, radius, Real.0)
            (z - center).modulus < Real.0
            modulus_nonneg(z - center)
            (z - center).modulus >= Real.0
            Real.0 <= (z - center).modulus
            lte_lt_trans(Real.0, (z - center).modulus, Real.0)
            Real.0 < Real.0
            false
        }
        not open_disk(center, radius).contains(z)
    }
}
