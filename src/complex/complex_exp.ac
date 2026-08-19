from nat import Nat, factorial_zero, factorial_one, factorial_step, pow_zero, pow_one, one_pow, semiring_zero_pow, alt_induction, add_sub, from_nat, lt_suc, lt_or_lte, lt_imp_lte_suc, lte_add_left, only_zero_lte_zero
from rat import Rat, nat_lte_imp_rat_lte, lt_some_nat, from_nat_mul, one_is_pos
from real import Real, converges, limit, converges_to, converges_to_imp_converges, converges_imp_converges_to,
    converges_to_unique, tail_bound, self_close, const_converges_to, const_limit, partial_suc, seq_lte,
    is_lower_bound, is_upper_bound, is_increasing, monotone_convergence_principle, increasing_convergent_bounded_by_limit, ub_imp_limit_lte,
    triangle_ineq, partial_nonneg, mul_seq, mul_seq_one, geom_converges, converges_mul_seq, comparison_test, partial_seq_lte,
    exp_term, two, two_nonzero, one_half_positive, suc_pos, mul_abs, abs_gte_zero, abs_not_neg,
    pos_imp_eq_abs, only_abs_zero_eq_zero, mul_inverse, mul_nonneg, mul_le_mul_pos_right, mul_le_mul_pos_left,
    div_le_of_mul_le, div_mul_cancel_right, mul_one_over, mul_div_left, lt_trans, lte_trans,
    add_lte_add, add_lt_lt, lt_add_right, lte_add_right, add_zero_left, from_rat_pos,
    from_rat_maintains_lt, from_rat_maintains_lte, from_nat_is_from_rat, rat_from_nat_add, from_nat_suc_pos_real,
    eps_lt_half, close_and_lt_imp_close, is_cauchy_seq, cauchy_imp_exists_limit, cauchy_imp_converges_to_limit,
    converges_to_has_tail_bound, tail_bound_implies_is_close, mul_from_rat, real_no_zero_divisors, cauchy_bound, abs_neg,
    prod_eq_to_div_eq, div_mul_cancel_left, lt_add_pos, rat_between_reals_gt, inverse_div, mul_div, one_half_plus_one_half, lte_abs, sub_both_eq_sub_add,
    div_cancel_common, mul_div_cancel, mul_inverse_comm, add_comm, add_assoc, neg_distrib, real_mul_comm, mul_neg_left,
    cauchy_coefficient, cauchy_product, cauchy_seq, cauchy_product_converges,
    absolutely_converges, abs_fn, absolutely_converges_imp_converges,
    sin_term, cos_term, sin_term_abs_converges, cos_term_abs_converges, two_mul_suc,
    pi, cos_pi_neg_one, sin_pi_zero,
    subsequence, converges_subsequence_double, choose_factorial_inverse
from order import lt_of_lte_of_lt, lt_of_lt_of_lte, lte_lt_trans, lt_lte_trans, lte_refl, lte_antisymm,
    lt_imp_lte, not_lte_imp_gt
from list import partial, partial_one, partial_pointwise_eq, partial_scalar_mul, partial_add
from data.basic.functions import function_extensionality
from algebra.ring.ring_hom import ring_hom_pow
from algebra.field.field import inverse_dist
from algebra.semigroup import mul_fn
from algebra.add_semigroup import add_fn
from complex.complex import Complex, re_add, im_add, re_zero, im_zero, re_one, im_one, re_from_real, im_from_real,
    div_one, mul_zero_left, div_from_real, inverse_lifts, from_real_one, re_mul, im_mul, eq_by_components,
    mul_comm, mul_assoc, distrib, from_real_neg, neg_one_lifts, re_neg, im_neg, real_mul_lifts, real_add_lifts, re_i, im_i, mul_one_left, mul_one_right, neg_eq_mul_neg_one, mul_from_real,
    i_squared
from complex.complex_abs import modulus_mul, modulus_from_real, modulus_of_one, modulus_nonneg, re_abs_le_modulus,
    im_abs_le_modulus, modulus_neg
from complex.complex_seq import re_seq, im_seq, complex_converges_to, complex_converges, complex_limit,
    componentwise_imp_complex_converges_to, complex_converges_to_imp_converges, complex_converges_to_imp_limit,
    complex_converges_imp_converges_to_limit, complex_limit_re, complex_limit_im, add_pointwise_converges_to, neg_pointwise_converges_to
from complex.complex_from_real_hom import complex_from_real_ring_hom, complex_from_real_ring_hom_hom
from comm_ring import binomial_term, binomial
from algebra.ring.ring import alternating_sign, alternating_sign_zero, alternating_sign_suc

/// The nth term of the complex exponential series: z^n / n!.
/// The factorial denominator is the real factorial embedded into the complex numbers,
/// and the quotient is complex division.
define complex_exp_term(z: Complex, n: Nat) -> Complex {
    z.pow(n) / Complex.from_real(Real.from_rat(Rat.from_nat(n.factorial)))
}

/// The complex exponential function z.exp = Σ z^n / n!,
/// defined as the limit of the partial sums of the complex exponential series.
define complex_exp(z: Complex) -> Complex {
    complex_limit(partial(complex_exp_term(z)))
}

/// The real number one is positive.
theorem real_one_positive {
    Real.1 > Real.0
} by {
    one_is_pos
    Rat.1.is_positive
    Rat.1 > Rat.0
    from_rat_pos(Rat.1)
    Real.from_rat(Rat.1).is_positive
    Real.from_rat(Rat.1) > Real.0
    Real.1 = Real.from_rat(Rat.1)
    Real.1 > Real.0
}

/// Two is positive.
theorem real_two_positive {
    two > Real.0
} by {
    two = Real.1 + Real.1
    real_one_positive
    Real.1 > Real.0
    lt_add_pos(Real.1, Real.1)
    Real.1 < Real.1 + Real.1
    Real.1 < two
    lt_trans(Real.0, Real.1, two)
    Real.0 < two
    two > Real.0
}

/// The absolute value of zero is zero.
theorem real_abs_zero {
    Real.0.abs = Real.0
} by {
    if Real.0.is_negative {
        -Real.0 = Real.0
    } else {
        Real.0 = Real.0
    }
}

/// A real number minus itself is zero.
theorem real_sub_self(a: Real) {
    a - a = Real.0
}

/// Adding the complex zero on the right fixes a complex number.
theorem complex_add_zero_right(a: Complex) {
    a + Complex.0 = a
}

/// The factorial, converted to a real number, is positive for every natural number.
theorem factorial_pos(n: Nat) {
    Real.from_rat(Rat.from_nat(n.factorial)) > Real.0
} by {
    Nat.1 <= n.factorial
    nat_lte_imp_rat_lte(Nat.1, n.factorial)
    Rat.1 <= Rat.from_nat(n.factorial)
    from_rat_maintains_lte(Rat.1, Rat.from_nat(n.factorial))
    Real.from_rat(Rat.1) <= Real.from_rat(Rat.from_nat(n.factorial))
    real_one_positive
    Real.1 > Real.0
    Real.1 = Real.from_rat(Rat.1)
    Real.from_rat(Rat.1) = Real.1
    Real.from_rat(Rat.1) > Real.0
    lt_of_lte_of_lt(Real.from_rat(Rat.1), Real.from_rat(Rat.from_nat(n.factorial)), Real.0)
    Real.from_rat(Rat.1) <= Real.from_rat(Rat.from_nat(n.factorial))
    Real.0 < Real.from_rat(Rat.1)
    Real.0 < Real.from_rat(Rat.from_nat(n.factorial))
    Real.from_rat(Rat.from_nat(n.factorial)) > Real.0
}

/// Cancellation of a nonzero factor on the left.
theorem real_mul_left_cancel(a: Real, b: Real, c: Real) {
    a != Real.0 and a * b = a * c implies b = c
} by {
    if a != Real.0 and a * b = a * c {
        mul_inverse(a)
        a * a.inverse = Real.1
        a.inverse * (a * b) = a.inverse * (a * c)
        a.inverse * (a * b) = (a.inverse * a) * b
        a.inverse * (a * c) = (a.inverse * a) * c
        a.inverse * a = Real.1
        Real.1 * b = b
        Real.1 * c = c
        b = c
    }
}

/// The absolute value of the inverse is the inverse of the absolute value.
theorem real_abs_inverse(b: Real) {
    b != Real.0 implies b.inverse.abs = b.abs.inverse
} by {
    if b != Real.0 {
        if b.abs = Real.0 {
            only_abs_zero_eq_zero(b)
            b = Real.0
            false
        }
        b.abs != Real.0
        mul_abs(b, b.inverse)
        (b * b.inverse).abs = b.abs * b.inverse.abs
        mul_inverse(b)
        b * b.inverse = Real.1
        (b * b.inverse).abs = Real.1.abs
        real_one_positive
        Real.1.is_positive
        pos_imp_eq_abs(Real.1)
        Real.1 = Real.1.abs
        Real.1.abs = Real.1
        b.abs * b.inverse.abs = Real.1
        mul_inverse(b.abs)
        b.abs * b.abs.inverse = Real.1
        real_mul_left_cancel(b.abs, b.inverse.abs, b.abs.inverse)
        b.inverse.abs = b.abs.inverse
    }
}

/// Natural-number powers in a monoid satisfy z^(n+1) = z * z^n.
theorem complex_pow_suc(z: Complex, n: Nat) {
    z.pow(n.suc) = z * z.pow(n)
}

/// Natural-number powers of zero vanish for positive exponents.
theorem complex_zero_pow_pos(n: Nat) {
    n >= Nat.1 implies Complex.0.pow(n) = Complex.0
} by {
    if n >= Nat.1 {
        add_sub(n, Nat.1)
        n - Nat.1 + Nat.1 = n
        (n - Nat.1).suc = n - Nat.1 + Nat.1
        (n - Nat.1).suc = n
        complex_pow_suc(Complex.0, n - Nat.1)
        Complex.0.pow((n - Nat.1).suc) = Complex.0 * Complex.0.pow(n - Nat.1)
        mul_zero_left(Complex.0.pow(n - Nat.1))
        Complex.0 * Complex.0.pow(n - Nat.1) = Complex.0
        Complex.0.pow((n - Nat.1).suc) = Complex.0
        Complex.0.pow(n) = Complex.0
    }
}

/// The modulus of a power is the power of the modulus: |z^n| = |z|^n.
theorem complex_modulus_pow(z: Complex, n: Nat) {
    z.pow(n).modulus = z.modulus.pow(n)
} by {
    define p(k: Nat) -> Bool {
        z.pow(k).modulus = z.modulus.pow(k)
    }
    z.pow(Nat.0) = Complex.1
    modulus_of_one
    Complex.1.modulus = Real.1
    pow_zero(z.modulus)
    z.modulus.pow(Nat.0) = Real.1
    z.pow(Nat.0).modulus = z.modulus.pow(Nat.0)
    p(Nat.0)
    forall(k: Nat) {
        if p(k) {
            complex_pow_suc(z, k)
            z.pow(k.suc) = z * z.pow(k)
            modulus_mul(z, z.pow(k))
            (z * z.pow(k)).modulus = z.modulus * z.pow(k).modulus
            z.pow(k.suc).modulus = z.modulus * z.pow(k).modulus
            z.pow(k).modulus = z.modulus.pow(k)
            z.modulus * z.pow(k).modulus = z.modulus * z.modulus.pow(k)
            z.modulus.pow(k.suc) = z.modulus * z.modulus.pow(k)
            z.pow(k.suc).modulus = z.modulus.pow(k.suc)
            p(k.suc)
        }
    }
    p(Nat.0) and forall(k: Nat) { p(k) implies p(k.suc) }
    alt_induction(p)
    p(n)
}

/// The modulus of the nth term of the complex exponential series is the real
/// exponential term at the modulus: |z^n / n!| = |z|^n / n!.
theorem complex_exp_term_modulus(z: Complex, n: Nat) {
    complex_exp_term(z, n).modulus = exp_term(z.modulus, n)
} by {
    let d: Real = Real.from_rat(Rat.from_nat(n.factorial))
    factorial_pos(n)
    d > Real.0
    d != Real.0
    d.is_positive
    pos_imp_eq_abs(d)
    d = d.abs
    d.abs = d

    complex_exp_term(z, n) = z.pow(n) / Complex.from_real(d)
    z.pow(n) / Complex.from_real(d) = z.pow(n) * Complex.from_real(d).inverse
    inverse_lifts(d)
    Complex.from_real(d).inverse = Complex.from_real(d.inverse)
    z.pow(n) * Complex.from_real(d).inverse = z.pow(n) * Complex.from_real(d.inverse)
    complex_exp_term(z, n) = z.pow(n) * Complex.from_real(d.inverse)

    complex_modulus_pow(z, n)
    z.pow(n).modulus = z.modulus.pow(n)
    modulus_mul(z.pow(n), Complex.from_real(d.inverse))
    (z.pow(n) * Complex.from_real(d.inverse)).modulus = z.pow(n).modulus * Complex.from_real(d.inverse).modulus
    modulus_from_real(d.inverse)
    Complex.from_real(d.inverse).modulus = d.inverse.abs
    real_abs_inverse(d)
    d.inverse.abs = d.abs.inverse
    d.inverse.abs = d.inverse
    (z.pow(n) * Complex.from_real(d.inverse)).modulus = z.modulus.pow(n) * d.inverse

    complex_exp_term(z, n).modulus = z.modulus.pow(n) * d.inverse
    z.modulus.pow(n) * d.inverse = z.modulus.pow(n) / d
    exp_term(z.modulus, n) = z.modulus.pow(n) / Real.from_rat(Rat.from_nat(n.factorial))
    z.modulus.pow(n) / d = exp_term(z.modulus, n)
    complex_exp_term(z, n).modulus = exp_term(z.modulus, n)
}

/// The zeroth term of the complex exponential series is one.
theorem complex_exp_term_zero_index(z: Complex) {
    complex_exp_term(z, Nat.0) = Complex.1
} by {
    complex_exp_term(z, Nat.0) = z.pow(Nat.0) / Complex.from_real(Real.from_rat(Rat.from_nat(Nat.0.factorial)))
    z.pow(Nat.0) = Complex.1
    Nat.0.factorial = Nat.1
    Rat.from_nat(Nat.1) = Rat.1
    Real.from_rat(Rat.1) = Real.1
    Complex.from_real(Real.1) = Complex.1
    Complex.1 / Complex.1 = Complex.1
}

/// The terms of the complex exponential series at zero vanish past the zeroth term.
theorem complex_exp_term_zero_vanishes(n: Nat) {
    n >= Nat.1 implies complex_exp_term(Complex.0, n) = Complex.0
} by {
    if n >= Nat.1 {
        complex_exp_term(Complex.0, n) = Complex.0.pow(n) / Complex.from_real(Real.from_rat(Rat.from_nat(n.factorial)))
        complex_zero_pow_pos(n)
        Complex.0.pow(n) = Complex.0
        Complex.0 / Complex.from_real(Real.from_rat(Rat.from_nat(n.factorial))) = Complex.0
        complex_exp_term(Complex.0, n) = Complex.0
    }
}

/// The imaginary part of every term of the complex exponential series at zero is zero.
theorem complex_exp_term_zero_im(k: Nat) {
    complex_exp_term(Complex.0, k).im = Real.0
} by {
    if k = Nat.0 {
        complex_exp_term_zero_index(Complex.0)
        complex_exp_term(Complex.0, Nat.0) = Complex.1
        complex_exp_term(Complex.0, k).im = Complex.1.im
        Complex.1.im = Real.0
        complex_exp_term(Complex.0, k).im = Real.0
    } else {
        k != Nat.0
        k >= Nat.1
        complex_exp_term_zero_vanishes(k)
        complex_exp_term(Complex.0, k) = Complex.0
        complex_exp_term(Complex.0, k).im = Complex.0.im
        Complex.0.im = Real.0
        complex_exp_term(Complex.0, k).im = Real.0
    }
}

/// The partial sums of the complex exponential series at zero are one from the first index on.
theorem complex_exp_zero_partial(k: Nat) {
    partial(complex_exp_term(Complex.0), k.suc) = Complex.1
} by {
    define p(x: Nat) -> Bool {
        partial(complex_exp_term(Complex.0), x.suc) = Complex.1
    }
    partial_one(complex_exp_term(Complex.0))
    partial(complex_exp_term(Complex.0), Nat.1) = complex_exp_term(Complex.0, Nat.0)
    complex_exp_term_zero_index(Complex.0)
    complex_exp_term(Complex.0, Nat.0) = Complex.1
    p(Nat.0)
    forall(x: Nat) {
        if p(x) {
            partial(complex_exp_term(Complex.0), x.suc) = Complex.1
            partial(complex_exp_term(Complex.0), x.suc.suc) = partial(complex_exp_term(Complex.0), x.suc) + complex_exp_term(Complex.0, x.suc)
            x.suc > Nat.0
            lt_imp_lte_suc(Nat.0, x.suc)
            Nat.0.suc <= x.suc
            Nat.0.suc = Nat.1
            Nat.1 <= x.suc
            x.suc >= Nat.1
            complex_exp_term_zero_vanishes(x.suc)
            x.suc >= Nat.1 implies complex_exp_term(Complex.0, x.suc) = Complex.0
            complex_exp_term(Complex.0, x.suc) = Complex.0
            partial(complex_exp_term(Complex.0), x.suc.suc) = Complex.1 + Complex.0
            complex_add_zero_right(Complex.1)
            Complex.1 + Complex.0 = Complex.1
            partial(complex_exp_term(Complex.0), x.suc.suc) = Complex.1
            p(x.suc)
        }
    }
    p(Nat.0) and forall(x: Nat) { p(x) implies p(x.suc) }
    alt_induction(p)
    p(k)
}

/// The partial sums of the complex exponential series at zero are one at every index
/// at least one.
theorem complex_exp_zero_partial_ge(i: Nat) {
    i >= Nat.1 implies partial(complex_exp_term(Complex.0), i) = Complex.1
} by {
    if i >= Nat.1 {
        add_sub(i, Nat.1)
        i - Nat.1 + Nat.1 = i
        (i - Nat.1).suc = i - Nat.1 + Nat.1
        (i - Nat.1).suc = i
        complex_exp_zero_partial(i - Nat.1)
        partial(complex_exp_term(Complex.0), (i - Nat.1).suc) = Complex.1
        partial(complex_exp_term(Complex.0), i) = Complex.1
    }
}

/// The zero partial sum of a complex sequence is the complex zero.
theorem complex_partial_zero(f: Nat -> Complex) {
    partial(f, Nat.0) = Complex.0
}

/// The successor partial sum of a complex sequence adds the next term.
theorem complex_partial_suc(f: Nat -> Complex, n: Nat) {
    partial(f, n.suc) = partial(f, n) + f(n)
}

/// The real part of a partial sum is the partial sum of the real parts.
theorem partial_re_seq(f: Nat -> Complex, n: Nat) {
    re_seq(partial(f), n) = partial(re_seq(f), n)
} by {
    define p(m: Nat) -> Bool {
        re_seq(partial(f), m) = partial(re_seq(f), m)
    }
    complex_partial_zero(f)
    partial(f, Nat.0) = Complex.0
    re_seq(partial(f), Nat.0) = Complex.0.re
    Complex.0.re = Real.0
    partial(re_seq(f), Nat.0) = Real.0
    p(Nat.0)
    forall(m: Nat) {
        if p(m) {
            re_seq(partial(f), m) = partial(re_seq(f), m)
            complex_partial_suc(f, m)
            partial(f, m.suc) = partial(f, m) + f(m)
            re_seq(f, m) = f(m).re
            partial_suc(re_seq(f), m)
            partial(re_seq(f), m.suc) = partial(re_seq(f), m) + re_seq(f, m)
            re_add(partial(f, m), f(m))
            (partial(f, m) + f(m)).re = partial(f, m).re + f(m).re
            re_seq(partial(f), m) = partial(f, m).re
            (partial(f, m) + f(m)).re = re_seq(partial(f), m) + re_seq(f, m)
            re_seq(partial(f), m.suc) = re_seq(partial(f), m) + re_seq(f, m)
            re_seq(partial(f), m.suc) = partial(re_seq(f), m) + re_seq(f, m)
            re_seq(partial(f), m.suc) = partial(re_seq(f), m.suc)
            p(m.suc)
        }
    }
    p(Nat.0) and forall(m: Nat) { p(m) implies p(m.suc) }
    alt_induction(p)
    p(n)
}

/// The imaginary part of a partial sum is the partial sum of the imaginary parts.
theorem partial_im_seq(f: Nat -> Complex, n: Nat) {
    im_seq(partial(f), n) = partial(im_seq(f), n)
} by {
    define p(m: Nat) -> Bool {
        im_seq(partial(f), m) = partial(im_seq(f), m)
    }
    complex_partial_zero(f)
    partial(f, Nat.0) = Complex.0
    im_seq(partial(f), Nat.0) = Complex.0.im
    Complex.0.im = Real.0
    partial(im_seq(f), Nat.0) = Real.0
    p(Nat.0)
    forall(m: Nat) {
        if p(m) {
            im_seq(partial(f), m) = partial(im_seq(f), m)
            complex_partial_suc(f, m)
            partial(f, m.suc) = partial(f, m) + f(m)
            im_seq(f, m) = f(m).im
            partial_suc(im_seq(f), m)
            partial(im_seq(f), m.suc) = partial(im_seq(f), m) + im_seq(f, m)
            im_add(partial(f, m), f(m))
            (partial(f, m) + f(m)).im = partial(f, m).im + f(m).im
            im_seq(partial(f), m) = partial(f, m).im
            (partial(f, m) + f(m)).im = im_seq(partial(f), m) + im_seq(f, m)
            im_seq(partial(f), m.suc) = im_seq(partial(f), m) + im_seq(f, m)
            im_seq(partial(f), m.suc) = partial(im_seq(f), m) + im_seq(f, m)
            im_seq(partial(f), m.suc) = partial(im_seq(f), m.suc)
            p(m.suc)
        }
    }
    p(Nat.0) and forall(m: Nat) { p(m) implies p(m.suc) }
    alt_induction(p)
    p(n)
}

/// A sequence that is one from the first index on converges to one.
theorem eventually_one_converges_to(q: Nat -> Real) {
    (forall(j: Nat) { j >= Nat.1 implies q(j) = Real.1 })
    implies
    converges_to(q, Real.1)
} by {
    forall(eps: Real) {
        if eps.is_positive {
            forall(i: Nat) {
                if Nat.1 <= i {
                    i >= Nat.1
                    q(i) = Real.1
                    self_close(Real.1, eps)
                    Real.1.is_close(Real.1, eps)
                    q(i).is_close(Real.1, eps)
                }
            }
            tail_bound(q, Real.1, Nat.1, eps)
        }
    }
}

/// The partial sums of the imaginary parts of the complex exponential series at zero vanish.
theorem complex_exp_zero_im_partial_zero(n: Nat) {
    partial(im_seq(complex_exp_term(Complex.0)), n) = Real.0
} by {
    define p(k: Nat) -> Bool {
        partial(im_seq(complex_exp_term(Complex.0)), k) = Real.0
    }
    partial(im_seq(complex_exp_term(Complex.0)), Nat.0) = Real.0
    p(Nat.0)
    forall(k: Nat) {
        if p(k) {
            partial(im_seq(complex_exp_term(Complex.0)), k) = Real.0
            partial_suc(im_seq(complex_exp_term(Complex.0)), k)
            partial(im_seq(complex_exp_term(Complex.0)), k.suc) = partial(im_seq(complex_exp_term(Complex.0)), k) + im_seq(complex_exp_term(Complex.0), k)
            im_seq(complex_exp_term(Complex.0), k) = complex_exp_term(Complex.0, k).im
            complex_exp_term_zero_im(k)
            complex_exp_term(Complex.0, k).im = Real.0
            im_seq(complex_exp_term(Complex.0), k) = Real.0
            partial(im_seq(complex_exp_term(Complex.0)), k.suc) = Real.0 + im_seq(complex_exp_term(Complex.0), k)
            partial(im_seq(complex_exp_term(Complex.0)), k.suc) = Real.0 + Real.0
            Real.0 + Real.0 = Real.0
            partial(im_seq(complex_exp_term(Complex.0)), k.suc) = Real.0
            p(k.suc)
        }
    }
    p(Nat.0) and forall(k: Nat) { p(k) implies p(k.suc) }
    alt_induction(p)
    p(n)
}

/// The partial sums of the constant zero sequence vanish.
theorem partial_zero_seq_const(n: Nat) {
    partial(constant[Nat, Real](Real.0), n) = Real.0
} by {
    define p(k: Nat) -> Bool {
        partial(constant[Nat, Real](Real.0), k) = Real.0
    }
    partial(constant[Nat, Real](Real.0), Nat.0) = Real.0
    p(Nat.0)
    forall(k: Nat) {
        if p(k) {
            partial(constant[Nat, Real](Real.0), k) = Real.0
            partial_suc(constant[Nat, Real](Real.0), k)
            partial(constant[Nat, Real](Real.0), k.suc) = partial(constant[Nat, Real](Real.0), k) + constant[Nat, Real](Real.0, k)
            constant[Nat, Real](Real.0, k) = Real.0
            partial(constant[Nat, Real](Real.0), k.suc) = Real.0 + constant[Nat, Real](Real.0, k)
            partial(constant[Nat, Real](Real.0), k.suc) = Real.0 + Real.0
            Real.0 + Real.0 = Real.0
            partial(constant[Nat, Real](Real.0), k.suc) = Real.0
            p(k.suc)
        }
    }
    p(Nat.0) and forall(k: Nat) { p(k) implies p(k.suc) }
    alt_induction(p)
    p(n)
}

/// The real-part sequence of the partial sums of the complex exponential series at zero
/// converges to one.
theorem complex_exp_zero_re_converges_to {
    converges_to(re_seq(partial(complex_exp_term(Complex.0))), Real.1)
} by {
    forall(i: Nat) {
        if i >= Nat.1 {
            complex_exp_zero_partial_ge(i)
            i >= Nat.1 implies partial(complex_exp_term(Complex.0), i) = Complex.1
            partial(complex_exp_term(Complex.0), i) = Complex.1
            partial(complex_exp_term(Complex.0), i).re = Complex.1.re
            re_seq(partial(complex_exp_term(Complex.0)), i) = Complex.1.re
            Complex.1.re = Real.1
            re_seq(partial(complex_exp_term(Complex.0)), i) = Real.1
        }
    }
    eventually_one_converges_to(re_seq(partial(complex_exp_term(Complex.0))))
    converges_to(re_seq(partial(complex_exp_term(Complex.0))), Real.1)
}

/// The imaginary-part sequence of the partial sums of the complex exponential series
/// at zero converges to zero.
theorem complex_exp_zero_im_converges_to {
    converges_to(im_seq(partial(complex_exp_term(Complex.0))), Real.0)
} by {
    forall(n: Nat) {
        partial_im_seq(complex_exp_term(Complex.0), n)
    }
    im_seq(partial(complex_exp_term(Complex.0))) = partial(im_seq(complex_exp_term(Complex.0)))
    forall(n: Nat) {
        complex_exp_zero_im_partial_zero(n)
        partial(im_seq(complex_exp_term(Complex.0)), n) = Real.0
        partial(im_seq(complex_exp_term(Complex.0)), n) = constant[Nat, Real](Real.0, n)
    }
    partial(im_seq(complex_exp_term(Complex.0))) = constant[Nat, Real](Real.0)
    im_seq(partial(complex_exp_term(Complex.0))) = constant[Nat, Real](Real.0)
    const_converges_to(Real.0)
    converges_to(constant[Nat, Real](Real.0), Real.0)
    converges_to(im_seq(partial(complex_exp_term(Complex.0))), Real.0)
}

/// The exponential of zero is one.
theorem complex_exp_zero {
    complex_exp(Complex.0) = Complex.1
} by {
    complex_exp_zero_re_converges_to
    complex_exp_zero_im_converges_to
    converges_to(re_seq(partial(complex_exp_term(Complex.0))), Real.1)
    converges_to(im_seq(partial(complex_exp_term(Complex.0))), Real.0)
    re_one
    im_one
    componentwise_imp_complex_converges_to(partial(complex_exp_term(Complex.0)), Complex.1)
    complex_converges_to(partial(complex_exp_term(Complex.0)), Complex.1)
    complex_converges_to_imp_limit(partial(complex_exp_term(Complex.0)), Complex.1)
    complex_limit(partial(complex_exp_term(Complex.0))) = Complex.1
    complex_exp(Complex.0) = complex_limit(partial(complex_exp_term(Complex.0)))
    complex_exp(Complex.0) = Complex.1
}

attributes Complex {
    /// The complex exponential function.
    let exp = complex_exp
}

// ---------------------------------------------------------------------------
// Real-series plumbing for the convergence of the complex exponential series.
// The ratio-test machinery of src/real/Real.exp.ac is private to the real package,
// so the needed lemmas are restated here using only the public real API.
// ---------------------------------------------------------------------------

/// Natural-number powers in a monoid satisfy x^(n+1) = x * x^n.
theorem real_pow_suc(x: Real, n: Nat) {
    x.pow(n.suc) = x * x.pow(n)
}

/// The absolute value of a power is the power of the absolute value.
theorem real_abs_pow(x: Real, n: Nat) {
    x.pow(n).abs = x.abs.pow(n)
} by {
    define p(k: Nat) -> Bool {
        x.pow(k).abs = x.abs.pow(k)
    }
    pow_zero(x)
    x.pow(Nat.0) = Real.1
    real_one_positive
    Real.1.is_positive
    pos_imp_eq_abs(Real.1)
    Real.1 = Real.1.abs
    Real.1.abs = Real.1
    pow_zero(x.abs)
    x.abs.pow(Nat.0) = Real.1
    p(Nat.0)
    forall(k: Nat) {
        if p(k) {
            real_pow_suc(x, k)
            x.pow(k.suc) = x * x.pow(k)
            mul_abs(x, x.pow(k))
            (x * x.pow(k)).abs = x.abs * x.pow(k).abs
            x.pow(k.suc).abs = x.abs * x.pow(k).abs
            x.pow(k).abs = x.abs.pow(k)
            x.abs * x.pow(k).abs = x.abs * x.abs.pow(k)
            real_pow_suc(x.abs, k)
            x.abs.pow(k.suc) = x.abs * x.abs.pow(k)
            x.pow(k.suc).abs = x.abs.pow(k.suc)
            p(k.suc)
        }
    }
    p(Nat.0) and forall(k: Nat) { p(k) implies p(k.suc) }
    alt_induction(p)
    p(n)
}

/// The absolute value of a quotient is the quotient of absolute values.
theorem real_abs_div(a: Real, b: Real) {
    b != Real.0 implies (a / b).abs = a.abs / b.abs
} by {
    if b != Real.0 {
        (a / b).abs = (a * b.inverse).abs
        mul_abs(a, b.inverse)
        (a * b.inverse).abs = a.abs * b.inverse.abs
        real_abs_inverse(b)
        b.inverse.abs = b.abs.inverse
        a.abs * b.inverse.abs = a.abs * b.abs.inverse
        a.abs * b.abs.inverse = a.abs / b.abs
        (a / b).abs = a.abs / b.abs
    }
}

/// The absolute value of an exponential term: |x^n / n!| = |x|^n / n!.
theorem exp_term_abs_value(x: Real, n: Nat) {
    exp_term(x, n).abs = x.abs.pow(n) / Real.from_rat(Rat.from_nat(n.factorial))
} by {
    let d: Real = Real.from_rat(Rat.from_nat(n.factorial))
    factorial_pos(n)
    d > Real.0
    d != Real.0
    exp_term(x, n) = x.pow(n) / d
    exp_term(x, n).abs = (x.pow(n) / d).abs
    real_abs_div(x.pow(n), d)
    (x.pow(n) / d).abs = x.pow(n).abs / d.abs
    real_abs_pow(x, n)
    x.pow(n).abs = x.abs.pow(n)
    x.pow(n).abs / d.abs = x.abs.pow(n) / d.abs
    d.is_positive
    pos_imp_eq_abs(d)
    d = d.abs
    d.abs = d
    x.pow(n).abs / d.abs = x.abs.pow(n) / d
    exp_term(x, n).abs = x.abs.pow(n) / d
}

/// Two times one half is one.
theorem two_mul_one_half {
    two * Real.one_half = Real.1
} by {
    two = Real.1 + Real.1
    Real.1 + Real.1 = Real.from_rat(Rat.1) + Real.from_rat(Rat.1)
    Real.1 = Real.from_rat(Rat.1)
    Real.1 + Real.1 = Real.from_rat(Rat.1 + Rat.1)
    Rat.1 + Rat.1 = Rat.2
    Real.1 + Real.1 = Real.from_rat(Rat.2)
    two = Real.from_rat(Rat.2)
    Real.one_half = Real.from_rat(Rat.2.inverse)
    two * Real.one_half = Real.from_rat(Rat.2) * Real.from_rat(Rat.2.inverse)
    mul_from_rat(Rat.2, Rat.2.inverse)
    Real.from_rat(Rat.2) * Real.from_rat(Rat.2.inverse) = Real.from_rat(Rat.2 * Rat.2.inverse)
    Rat.2 * Rat.2.inverse = Rat.1
    Real.from_rat(Rat.2 * Rat.2.inverse) = Real.from_rat(Rat.1)
    Real.from_rat(Rat.1) = Real.1
    Real.from_rat(Rat.2 * Rat.2.inverse) = Real.1
    two * Real.one_half = Real.1
}

/// Dividing by two is multiplying by one half.
theorem div_two_is_mul_half(a: Real) {
    a / two = a * Real.one_half
} by {
    two != Real.0
    mul_inverse(two)
    two * two.inverse = Real.1
    two_mul_one_half
    two * Real.one_half = Real.1
    real_mul_left_cancel(two, two.inverse, Real.one_half)
    two.inverse = Real.one_half
    a / two = a * two.inverse
    a * two.inverse = a * Real.one_half
    a / two = a * Real.one_half
}

/// Natural-number powers of a nonnegative real are nonnegative.
theorem real_pow_nonneg(r: Real, n: Nat) {
    r >= Real.0 implies r.pow(n) >= Real.0
} by {
    define p(k: Nat) -> Bool {
        r >= Real.0 implies r.pow(k) >= Real.0
    }
    if r >= Real.0 {
        pow_zero(r)
        r.pow(Nat.0) = Real.1
        real_one_positive
        Real.1 > Real.0
        Real.1 >= Real.0
        r.pow(Nat.0) >= Real.0
    }
    p(Nat.0)
    forall(k: Nat) {
        if p(k) {
            if r >= Real.0 {
                r.pow(k) >= Real.0
                real_pow_suc(r, k)
                r.pow(k.suc) = r * r.pow(k)
                mul_nonneg(r, r.pow(k))
                r * r.pow(k) >= Real.0
                r.pow(k.suc) >= Real.0
            }
            p(k.suc)
        }
    }
    p(Nat.0) and forall(k: Nat) { p(k) implies p(k.suc) }
    alt_induction(p)
    p(n)
}

/// Multiplication form of the exp_term recurrence:
/// exp_term(x, n+1) * (n+1) = x * exp_term(x, n).
theorem exp_term_mul_recurrence(x: Real, n: Nat) {
    exp_term(x, n.suc) * Real.from_rat(Rat.from_nat(n.suc)) = x * exp_term(x, n)
} by {
    exp_term(x, n.suc) = x.pow(n.suc) / Real.from_rat(Rat.from_nat(n.suc.factorial))
    exp_term(x, n) = x.pow(n) / Real.from_rat(Rat.from_nat(n.factorial))

    let f_n = Real.from_rat(Rat.from_nat(n.factorial))
    let f_n_suc = Real.from_rat(Rat.from_nat(n.suc.factorial))
    let n_plus_1 = Real.from_rat(Rat.from_nat(n.suc))

    real_pow_suc(x, n)
    x.pow(n.suc) = x * x.pow(n)
    factorial_step(n)
    n.suc.factorial = n.suc * n.factorial
    from_nat_mul(n.suc, n.factorial)
    Rat.from_nat(n.suc) * Rat.from_nat(n.factorial) = Rat.from_nat(n.suc * n.factorial)
    mul_from_rat(Rat.from_nat(n.suc), Rat.from_nat(n.factorial))
    Real.from_rat(Rat.from_nat(n.suc)) * Real.from_rat(Rat.from_nat(n.factorial)) =
        Real.from_rat(Rat.from_nat(n.suc) * Rat.from_nat(n.factorial))
    f_n_suc = n_plus_1 * f_n

    factorial_pos(n)
    f_n > Real.0
    f_n != Real.0
    suc_pos(n)
    n_plus_1 > Real.0
    n_plus_1 != Real.0
    factorial_pos(n.suc)
    f_n_suc != Real.0

    exp_term(x, n.suc) * n_plus_1 = (x.pow(n.suc) / f_n_suc) * n_plus_1
    (x.pow(n.suc) / f_n_suc) * n_plus_1 = ((x * x.pow(n)) / (n_plus_1 * f_n)) * n_plus_1

    n_plus_1 * f_n = f_n * n_plus_1
    f_n * (((x * x.pow(n)) / (n_plus_1 * f_n)) * n_plus_1) = ((x * x.pow(n)) / (n_plus_1 * f_n)) * (f_n * n_plus_1)
    f_n_suc * ((x * x.pow(n)) / f_n_suc) = x * x.pow(n)

    n_plus_1 / Real.1 = n_plus_1
    mul_div(x * x.pow(n), n_plus_1 * f_n, n_plus_1, Real.1)
    ((x * x.pow(n)) / (n_plus_1 * f_n)) * (n_plus_1 / Real.1) =
        ((x * x.pow(n)) * n_plus_1) / ((n_plus_1 * f_n) * Real.1)
    ((x * x.pow(n)) / (n_plus_1 * f_n)) * n_plus_1 =
        ((x * x.pow(n)) * n_plus_1) / ((n_plus_1 * f_n) * Real.1)
    ((x * x.pow(n)) * n_plus_1) / ((n_plus_1 * f_n) * Real.1) =
        ((x * x.pow(n)) * n_plus_1) / (n_plus_1 * f_n)
    ((x * x.pow(n)) * n_plus_1) / (n_plus_1 * f_n) =
        ((x * x.pow(n)) * n_plus_1) / (f_n * n_plus_1)
    div_cancel_common(x * x.pow(n), n_plus_1, f_n)
    ((x * x.pow(n)) * n_plus_1) / (f_n * n_plus_1) = (x * x.pow(n)) / f_n
    ((x * x.pow(n)) / (n_plus_1 * f_n)) * n_plus_1 = (x * x.pow(n)) / f_n

    x * exp_term(x, n) = x * (x.pow(n) / f_n)
    x * (x.pow(n) / f_n) = (x * x.pow(n)) / f_n

    exp_term(x, n.suc) * n_plus_1 = (x * x.pow(n)) / f_n
    x * exp_term(x, n) = (x * x.pow(n)) / f_n
    exp_term(x, n.suc) * n_plus_1 = x * exp_term(x, n)
    exp_term(x, n.suc) * Real.from_rat(Rat.from_nat(n.suc)) = x * exp_term(x, n)
}

/// Absolute-value form of the exp_term recurrence:
/// |exp_term(x, n+1)| * (n+1) = |x| * |exp_term(x, n)|.
theorem exp_term_mul_recurrence_abs(x: Real, n: Nat) {
    exp_term(x, n.suc).abs * Real.from_rat(Rat.from_nat(n.suc)) = x.abs * exp_term(x, n).abs
} by {
    let n_plus_1 = Real.from_rat(Rat.from_nat(n.suc))
    exp_term_mul_recurrence(x, n)
    exp_term(x, n.suc) * n_plus_1 = x * exp_term(x, n)

    (exp_term(x, n.suc) * n_plus_1).abs = (x * exp_term(x, n)).abs

    mul_abs(exp_term(x, n.suc), n_plus_1)
    (exp_term(x, n.suc) * n_plus_1).abs = exp_term(x, n.suc).abs * n_plus_1.abs
    mul_abs(x, exp_term(x, n))
    (x * exp_term(x, n)).abs = x.abs * exp_term(x, n).abs

    suc_pos(n)
    n_plus_1 > Real.0
    n_plus_1.is_positive
    pos_imp_eq_abs(n_plus_1)
    n_plus_1 = n_plus_1.abs
    n_plus_1.abs = n_plus_1

    exp_term(x, n.suc).abs * n_plus_1 = x.abs * exp_term(x, n).abs
}

/// If an exponential term vanishes, so does its successor.
theorem exp_term_abs_zero_imp_suc_zero(x: Real, n: Nat) {
    exp_term(x, n).abs = Real.0 implies exp_term(x, n.suc).abs = Real.0
} by {
    if exp_term(x, n).abs = Real.0 {
        exp_term_mul_recurrence_abs(x, n)
        exp_term(x, n.suc).abs * Real.from_rat(Rat.from_nat(n.suc)) = x.abs * exp_term(x, n).abs
        x.abs * exp_term(x, n).abs = Real.0
        exp_term(x, n.suc).abs * Real.from_rat(Rat.from_nat(n.suc)) = Real.0
        suc_pos(n)
        Real.from_rat(Rat.from_nat(n.suc)) > Real.0
        Real.from_rat(Rat.from_nat(n.suc)) != Real.0
        real_no_zero_divisors(exp_term(x, n.suc).abs, Real.from_rat(Rat.from_nat(n.suc)))
        exp_term(x, n.suc).abs = Real.0 or Real.from_rat(Rat.from_nat(n.suc)) = Real.0
        exp_term(x, n.suc).abs = Real.0
    }
}

/// The ratio of absolute values of consecutive exp_term values.
theorem exp_term_ratio(x: Real, n: Nat) {
    exp_term(x, n).abs != Real.0 implies
    exp_term(x, n.suc).abs / exp_term(x, n).abs = x.abs / Real.from_rat(Rat.from_nat(n.suc))
} by {
    if exp_term(x, n).abs != Real.0 {
        let n_plus_1 = Real.from_rat(Rat.from_nat(n.suc))

        exp_term_mul_recurrence_abs(x, n)
        exp_term(x, n.suc).abs * n_plus_1 = x.abs * exp_term(x, n).abs

        suc_pos(n)
        n_plus_1 > Real.0
        n_plus_1.is_positive
        n_plus_1 != Real.0

        exp_term(x, n).abs != Real.0

        prod_eq_to_div_eq(exp_term(x, n.suc).abs, n_plus_1, x.abs, exp_term(x, n).abs)
        exp_term(x, n.suc).abs / exp_term(x, n).abs = x.abs / n_plus_1
        exp_term(x, n.suc).abs / exp_term(x, n).abs = x.abs / Real.from_rat(Rat.from_nat(n.suc))
    }
}

/// If 2|x| <= n+1, consecutive absolute terms shrink by a factor of at most one half.
theorem exp_term_ratio_le_half(x: Real, n: Nat) {
    exp_term(x, n).abs != Real.0 and x.abs * two <= Real.from_rat(Rat.from_nat(n.suc))
    implies exp_term(x, n.suc).abs <= Real.one_half * exp_term(x, n).abs
} by {
    if exp_term(x, n).abs != Real.0 and x.abs * two <= Real.from_rat(Rat.from_nat(n.suc)) {
        let n_plus_1 = Real.from_rat(Rat.from_nat(n.suc))
        exp_term_ratio(x, n)
        exp_term(x, n.suc).abs / exp_term(x, n).abs = x.abs / n_plus_1

        real_two_positive
        two > Real.0
        two != Real.0
        div_le_of_mul_le(x.abs, two, n_plus_1)
        x.abs <= n_plus_1 / two
        div_two_is_mul_half(n_plus_1)
        n_plus_1 / two = n_plus_1 * Real.one_half
        x.abs <= n_plus_1 * Real.one_half

        n_plus_1 > Real.0
        n_plus_1 != Real.0
        x.abs / n_plus_1 = x.abs * n_plus_1.inverse
        (x.abs * n_plus_1.inverse) * n_plus_1 = x.abs * (n_plus_1.inverse * n_plus_1)
        n_plus_1.inverse * n_plus_1 = Real.1
        x.abs * (n_plus_1.inverse * n_plus_1) = x.abs * Real.1
        x.abs * Real.1 = x.abs
        (x.abs / n_plus_1) * n_plus_1 = x.abs
        (x.abs / n_plus_1) * n_plus_1 <= n_plus_1 * Real.one_half
        div_le_of_mul_le(x.abs / n_plus_1, n_plus_1, n_plus_1 * Real.one_half)
        x.abs / n_plus_1 <= (n_plus_1 * Real.one_half) / n_plus_1
        div_mul_cancel_left(n_plus_1, Real.one_half)
        (n_plus_1 * Real.one_half) / n_plus_1 = Real.one_half
        x.abs / n_plus_1 <= Real.one_half

        exp_term(x, n).abs > Real.0
        mul_le_mul_pos_right(x.abs / n_plus_1, Real.one_half, exp_term(x, n).abs)
        (x.abs / n_plus_1) * exp_term(x, n).abs <= Real.one_half * exp_term(x, n).abs

        exp_term(x, n.suc).abs / exp_term(x, n).abs = x.abs / n_plus_1
        exp_term(x, n).abs != Real.0
        mul_div_cancel(exp_term(x, n.suc).abs, exp_term(x, n).abs)
        exp_term(x, n).abs * (exp_term(x, n.suc).abs / exp_term(x, n).abs) = exp_term(x, n.suc).abs
        exp_term(x, n).abs * (x.abs / n_plus_1) = exp_term(x, n.suc).abs
        (x.abs / n_plus_1) * exp_term(x, n).abs = exp_term(x, n.suc).abs
        exp_term(x, n.suc).abs = (x.abs / n_plus_1) * exp_term(x, n).abs
        exp_term(x, n.suc).abs <= Real.one_half * exp_term(x, n).abs
    }
}

/// The Archimedean property: every positive real number is below some natural number.
theorem exists_nat_gt_real(x: Real) {
    x.is_positive implies exists(n: Nat) {
        from_nat[Real](n) > x
    }
} by {
    if x.is_positive {
        let x_plus_one: Real = x + Real.1
        x_plus_one > x
        rat_between_reals_gt(x_plus_one, x)
        let r: Rat satisfy {
            x_plus_one > Real.from_rat(r) and Real.from_rat(r) > x
        }
        Real.from_rat(r) > x
        lt_some_nat(r)
        let n: Nat satisfy {
            r < Rat.from_nat(n)
        }
        r < Rat.from_nat(n)
        from_rat_maintains_lt(r, Rat.from_nat(n))
        Real.from_rat(r) < Real.from_rat(Rat.from_nat(n))
        from_nat_is_from_rat(n)
        Real.from_rat(Rat.from_nat(n)) = from_nat[Real](n)
        Real.from_rat(r) < from_nat[Real](n)
        lt_trans(x, Real.from_rat(r), from_nat[Real](n))
        x < from_nat[Real](n)
        from_nat[Real](n) > x
    }
}

/// For every real number, there is a natural number n with 2|x| <= n + 1.
theorem exists_ratio_index(x: Real) {
    exists(big_n: Nat) {
        x.abs * two <= Real.from_rat(Rat.from_nat(big_n + Nat.1))
    }
} by {
    let two_x_plus_one: Real = two * x.abs + Real.1
    real_two_positive
    two > Real.0
    lt_imp_lte(Real.0, two)
    two >= Real.0
    abs_gte_zero(x)
    x.abs >= Real.0
    mul_nonneg(two, x.abs)
    two * x.abs >= Real.0
    Real.0 <= two * x.abs
    lte_refl(Real.1)
    Real.1 <= Real.1
    add_lte_add(Real.1, Real.1, Real.0, two * x.abs)
    Real.1 + Real.0 <= Real.1 + two * x.abs
    Real.1 + Real.0 = Real.1
    Real.1 <= two * x.abs + Real.1
    real_one_positive
    Real.1 > Real.0
    Real.1 <= two * x.abs + Real.1
    Real.1 <= two_x_plus_one
    lt_of_lt_of_lte(Real.0, Real.1, two_x_plus_one)
    Real.0 < two_x_plus_one
    two_x_plus_one > Real.0
    two_x_plus_one.is_positive
    exists_nat_gt_real(two_x_plus_one)
    let m: Nat satisfy {
        from_nat[Real](m) > two_x_plus_one
    }
    from_nat[Real](m) > two * x.abs + Real.1
    lt_add_pos(two * x.abs, Real.1)
    two * x.abs < two * x.abs + Real.1
    lt_trans(two * x.abs, two * x.abs + Real.1, from_nat[Real](m))
    two * x.abs < from_nat[Real](m)
    Real.1 <= two_x_plus_one
    lt_of_lte_of_lt(Real.1, two_x_plus_one, from_nat[Real](m))
    Real.1 < from_nat[Real](m)
    from_nat[Real](m) > Real.1
    if m <= Nat.1 {
        nat_lte_imp_rat_lte(m, Nat.1)
        Rat.from_nat(m) <= Rat.from_nat(Nat.1)
        from_rat_maintains_lte(Rat.from_nat(m), Rat.from_nat(Nat.1))
        Real.from_rat(Rat.from_nat(m)) <= Real.from_rat(Rat.from_nat(Nat.1))
        from_nat_is_from_rat(m)
        from_nat_is_from_rat(Nat.1)
        from_nat[Real](m) <= from_nat[Real](Nat.1)
        from_nat[Real](Nat.1) = Real.1
        from_nat[Real](m) <= Real.1
        false
    }
    not (m <= Nat.1)
    lt_or_lte(Nat.1, m)
    Nat.1 < m
    lt_imp_lte_suc(Nat.1, m)
    Nat.1.suc <= m
    Nat.1.suc = Nat.2
    Nat.2 <= m
    m >= Nat.2
    Nat.1 <= m
    add_sub(m, Nat.1)
    m - Nat.1 + Nat.1 = m
    (m - Nat.1) + Nat.1 = m
    Real.from_rat(Rat.from_nat((m - Nat.1) + Nat.1)) = Real.from_rat(Rat.from_nat(m))
    from_nat_is_from_rat((m - Nat.1) + Nat.1)
    from_nat_is_from_rat(m)
    from_nat[Real]((m - Nat.1) + Nat.1) = from_nat[Real](m)
    two * x.abs < from_nat[Real]((m - Nat.1) + Nat.1)
    lt_imp_lte(two * x.abs, from_nat[Real]((m - Nat.1) + Nat.1))
    two * x.abs <= from_nat[Real]((m - Nat.1) + Nat.1)
    x.abs * two <= from_nat[Real]((m - Nat.1) + Nat.1)
}

/// The tail of the exponential series from n0 is geometrically dominated.
theorem exp_term_tail_geom_bound(x: Real, n0: Nat, j: Nat) {
    x.abs * two <= Real.from_rat(Rat.from_nat(n0 + Nat.1))
    implies exp_term(x, n0 + j).abs <= exp_term(x, n0).abs * Real.one_half.pow(j)
} by {
    define p(k: Nat) -> Bool {
        x.abs * two <= Real.from_rat(Rat.from_nat(n0 + Nat.1))
        implies exp_term(x, n0 + k).abs <= exp_term(x, n0).abs * Real.one_half.pow(k)
    }
    pow_zero(Real.one_half)
    Real.one_half.pow(Nat.0) = Real.1
    if x.abs * two <= Real.from_rat(Rat.from_nat(n0 + Nat.1)) {
        n0 + Nat.0 = n0
        exp_term(x, n0 + Nat.0) = exp_term(x, n0)
        exp_term(x, n0).abs * Real.one_half.pow(Nat.0) = exp_term(x, n0).abs
        exp_term(x, n0 + Nat.0).abs <= exp_term(x, n0).abs * Real.one_half.pow(Nat.0)
    }
    p(Nat.0)
    forall(k: Nat) {
        if p(k) {
            if x.abs * two <= Real.from_rat(Rat.from_nat(n0 + Nat.1)) {
                exp_term(x, n0 + k).abs <= exp_term(x, n0).abs * Real.one_half.pow(k)
                lt_suc(k)
                Nat.0 < k.suc
                lt_imp_lte_suc(Nat.0, k.suc)
                Nat.0.suc <= k.suc
                Nat.0.suc = Nat.1
                Nat.1 <= k.suc
                lte_add_left(Nat.1, k.suc, n0)
                n0 + Nat.1 <= n0 + k.suc
                (n0 + k).suc = n0 + k.suc
                n0 + Nat.1 <= (n0 + k).suc
                nat_lte_imp_rat_lte(n0 + Nat.1, (n0 + k).suc)
                Rat.from_nat(n0 + Nat.1) <= Rat.from_nat((n0 + k).suc)
                from_rat_maintains_lte(Rat.from_nat(n0 + Nat.1), Rat.from_nat((n0 + k).suc))
                Real.from_rat(Rat.from_nat(n0 + Nat.1)) <= Real.from_rat(Rat.from_nat((n0 + k).suc))
                from_nat_is_from_rat(n0 + Nat.1)
                from_nat_is_from_rat((n0 + k).suc)
                from_nat[Real](n0 + Nat.1) <= from_nat[Real]((n0 + k).suc)
                lte_trans(x.abs * two, Real.from_rat(Rat.from_nat(n0 + Nat.1)), Real.from_rat(Rat.from_nat((n0 + k).suc)))
                x.abs * two <= Real.from_rat(Rat.from_nat((n0 + k).suc))

                if exp_term(x, n0 + k).abs = Real.0 {
                    exp_term_abs_zero_imp_suc_zero(x, n0 + k)
                    exp_term(x, (n0 + k).suc).abs = Real.0
                    (n0 + k).suc = n0 + k.suc
                    exp_term(x, n0 + k.suc).abs = Real.0
                    abs_gte_zero(exp_term(x, n0))
                    exp_term(x, n0).abs >= Real.0
                    one_half_positive
                    Real.0 < Real.one_half
                    lt_imp_lte(Real.0, Real.one_half)
                    Real.one_half >= Real.0
                    real_pow_nonneg(Real.one_half, k.suc)
                    Real.one_half.pow(k.suc) >= Real.0
                    mul_nonneg(exp_term(x, n0).abs, Real.one_half.pow(k.suc))
                    exp_term(x, n0).abs * Real.one_half.pow(k.suc) >= Real.0
                    Real.0 <= exp_term(x, n0).abs * Real.one_half.pow(k.suc)
                    exp_term(x, n0 + k.suc).abs <= exp_term(x, n0).abs * Real.one_half.pow(k.suc)
                } else {
                    exp_term(x, n0 + k).abs != Real.0
                    exp_term_ratio_le_half(x, n0 + k)
                    exp_term(x, (n0 + k).suc).abs <= Real.one_half * exp_term(x, n0 + k).abs
                    (n0 + k).suc = n0 + k.suc
                    exp_term(x, n0 + k.suc).abs <= Real.one_half * exp_term(x, n0 + k).abs

                    one_half_positive
                    Real.one_half > Real.0
                    mul_le_mul_pos_right(exp_term(x, n0 + k).abs, exp_term(x, n0).abs * Real.one_half.pow(k), Real.one_half)
                    exp_term(x, n0 + k).abs * Real.one_half <= (exp_term(x, n0).abs * Real.one_half.pow(k)) * Real.one_half
                    Real.one_half * exp_term(x, n0 + k).abs = exp_term(x, n0 + k).abs * Real.one_half
                    exp_term(x, n0 + k.suc).abs <= exp_term(x, n0 + k).abs * Real.one_half
                    lte_trans(exp_term(x, n0 + k.suc).abs, exp_term(x, n0 + k).abs * Real.one_half, (exp_term(x, n0).abs * Real.one_half.pow(k)) * Real.one_half)
                    exp_term(x, n0 + k.suc).abs <= (exp_term(x, n0).abs * Real.one_half.pow(k)) * Real.one_half
                    (exp_term(x, n0).abs * Real.one_half.pow(k)) * Real.one_half = exp_term(x, n0).abs * (Real.one_half.pow(k) * Real.one_half)
                    Real.one_half.pow(k) * Real.one_half = Real.one_half * Real.one_half.pow(k)
                    real_pow_suc(Real.one_half, k)
                    Real.one_half.pow(k.suc) = Real.one_half * Real.one_half.pow(k)
                    Real.one_half.pow(k) * Real.one_half = Real.one_half.pow(k.suc)
                    (exp_term(x, n0).abs * Real.one_half.pow(k)) * Real.one_half = exp_term(x, n0).abs * Real.one_half.pow(k.suc)
                    exp_term(x, n0 + k.suc).abs <= exp_term(x, n0).abs * Real.one_half.pow(k.suc)
                }
            }
            p(k.suc)
        }
    }
    p(Nat.0) and forall(k: Nat) { p(k) implies p(k.suc) }
    alt_induction(p)
    p(j)
}

// ---------------------------------------------------------------------------
// Convergence of the real exponential series and of the complex exponential
// series, using only the public real API.
// ---------------------------------------------------------------------------

/// The pointwise absolute value of a real sequence.
define seq_abs(a: Nat -> Real, n: Nat) -> Real {
    a(n).abs
}

/// The tail of a sequence, starting at index n.
define seq_tail(a: Nat -> Real, n: Nat, i: Nat) -> Real {
    a(n + i)
}

/// The absolute value of a tail is the tail of the absolute values.
theorem seq_abs_tail_comm(a: Nat -> Real, n: Nat) {
    seq_abs(seq_tail(a, n)) = seq_tail(seq_abs(a), n)
} by {
    forall(i: Nat) {
        seq_abs(seq_tail(a, n), i) = seq_tail(a, n, i).abs
        seq_tail(a, n, i) = a(n + i)
        seq_tail(a, n, i).abs = a(n + i).abs
        seq_abs(a, n + i) = a(n + i).abs
        seq_tail(seq_abs(a), n, i) = seq_abs(a, n + i)
        seq_abs(seq_tail(a, n), i) = seq_tail(seq_abs(a), n, i)
    }
}

/// The partial sum of a length-n tail: partial(a, m + n) = partial(a, m) + partial(tail, n).
theorem partial_tail_decomp(a: Nat -> Real, m: Nat, n: Nat) {
    partial(a, m + n) = partial(a, m) + partial(seq_tail(a, m), n)
} by {
    define p(k: Nat) -> Bool {
        partial(a, m + k) = partial(a, m) + partial(seq_tail(a, m), k)
    }
    m + Nat.0 = m
    partial(seq_tail(a, m), Nat.0) = Real.0
    partial(a, m) + partial(seq_tail(a, m), Nat.0) = partial(a, m) + Real.0
    partial(a, m) + Real.0 = partial(a, m)
    partial(a, m + Nat.0) = partial(a, m) + partial(seq_tail(a, m), Nat.0)
    p(Nat.0)
    forall(k: Nat) {
        if p(k) {
            partial(a, m + k) = partial(a, m) + partial(seq_tail(a, m), k)
            m + k.suc = (m + k).suc
            partial_suc(a, m + k)
            partial(a, (m + k).suc) = partial(a, m + k) + a(m + k)
            partial(a, m + k.suc) = partial(a, m + k) + a(m + k)
            seq_tail(a, m, k) = a(m + k)
            partial_suc(seq_tail(a, m), k)
            partial(seq_tail(a, m), k.suc) = partial(seq_tail(a, m), k) + seq_tail(a, m, k)
            partial(seq_tail(a, m), k.suc) = partial(seq_tail(a, m), k) + a(m + k)
            partial(a, m) + partial(seq_tail(a, m), k.suc) = partial(a, m) + (partial(seq_tail(a, m), k) + a(m + k))
            partial(a, m + k.suc) = partial(a, m) + partial(seq_tail(a, m), k.suc)
            p(k.suc)
        }
    }
    p(Nat.0) and forall(k: Nat) { p(k) implies p(k.suc) }
    alt_induction(p)
    p(n)
}

/// The absolute difference of partial sums is bounded by the difference of the
/// partial sums of the absolute values.
theorem partial_diff_abs_le(a: Nat -> Real, i: Nat, j: Nat) {
    j <= i implies (partial(a, i) - partial(a, j)).abs <= partial(seq_abs(a), i) - partial(seq_abs(a), j)
} by {
    define p(m: Nat) -> Bool {
        forall(k: Nat) {
            k <= m implies (partial(a, m) - partial(a, k)).abs <= partial(seq_abs(a), m) - partial(seq_abs(a), k)
        }
    }
    forall(k: Nat) {
        if k <= Nat.0 {
            only_zero_lte_zero(k)
            k = Nat.0
            partial(a, Nat.0) = Real.0
            partial(a, k) = Real.0
            partial(a, Nat.0) - partial(a, k) = Real.0
            real_abs_zero
            (partial(a, Nat.0) - partial(a, k)).abs = Real.0
            partial(seq_abs(a), Nat.0) = Real.0
            partial(seq_abs(a), k) = Real.0
            partial(seq_abs(a), Nat.0) - partial(seq_abs(a), k) = Real.0
            (partial(a, Nat.0) - partial(a, k)).abs <= partial(seq_abs(a), Nat.0) - partial(seq_abs(a), k)
        }
    }
    p(Nat.0)
    forall(m: Nat) {
        if p(m) {
            forall(k: Nat) {
                if k <= m.suc {
                    if k <= m {
                        k <= m implies (partial(a, m) - partial(a, k)).abs <= partial(seq_abs(a), m) - partial(seq_abs(a), k)
                        (partial(a, m) - partial(a, k)).abs <= partial(seq_abs(a), m) - partial(seq_abs(a), k)
                        partial_suc(a, m)
                        partial(a, m.suc) = partial(a, m) + a(m)
                        partial(a, m.suc) - partial(a, k) = (partial(a, m) - partial(a, k)) + a(m)
                        triangle_ineq(partial(a, m) - partial(a, k), a(m))
                        (partial(a, m.suc) - partial(a, k)).abs <= (partial(a, m) - partial(a, k)).abs + a(m).abs
                        seq_abs(a, m) = a(m).abs
                        (partial(a, m.suc) - partial(a, k)).abs <= (partial(a, m) - partial(a, k)).abs + seq_abs(a, m)
                        lte_add_right((partial(a, m) - partial(a, k)).abs, partial(seq_abs(a), m) - partial(seq_abs(a), k), seq_abs(a, m))
                        (partial(a, m) - partial(a, k)).abs + seq_abs(a, m) <= (partial(seq_abs(a), m) - partial(seq_abs(a), k)) + seq_abs(a, m)
                        partial_suc(seq_abs(a), m)
                        partial(seq_abs(a), m.suc) = partial(seq_abs(a), m) + seq_abs(a, m)
                        partial(seq_abs(a), m.suc) - partial(seq_abs(a), k) = (partial(seq_abs(a), m) - partial(seq_abs(a), k)) + seq_abs(a, m)
                        lte_trans((partial(a, m.suc) - partial(a, k)).abs, (partial(a, m) - partial(a, k)).abs + seq_abs(a, m), partial(seq_abs(a), m.suc) - partial(seq_abs(a), k))
                        (partial(a, m.suc) - partial(a, k)).abs <= partial(seq_abs(a), m.suc) - partial(seq_abs(a), k)
                    } else {
                        not (k <= m)
                        lt_or_lte(m, k)
                        m < k or k <= m
                        m < k
                        lt_imp_lte_suc(m, k)
                        m.suc <= k
                        k <= m.suc
                        lte_antisymm(k, m.suc)
                        k = m.suc
                        partial(a, m.suc) - partial(a, k) = Real.0
                        real_abs_zero
                        (partial(a, m.suc) - partial(a, k)).abs = Real.0
                        partial(seq_abs(a), m.suc) - partial(seq_abs(a), k) = Real.0
                        (partial(a, m.suc) - partial(a, k)).abs <= partial(seq_abs(a), m.suc) - partial(seq_abs(a), k)
                    }
                }
            }
            p(m.suc)
        }
    }
    p(Nat.0) and forall(m: Nat) { p(m) implies p(m.suc) }
    alt_induction(p)
    p(i)
    forall(k: Nat) {
        k <= i implies (partial(a, i) - partial(a, k)).abs <= partial(seq_abs(a), i) - partial(seq_abs(a), k)
    }
    if j <= i {
        (partial(a, i) - partial(a, j)).abs <= partial(seq_abs(a), i) - partial(seq_abs(a), j)
    }
}

/// If the series of absolute values converges, then the series converges.
theorem abs_series_imp_series(a: Nat -> Real) {
    converges(partial(seq_abs(a))) implies converges(partial(a))
} by {
    if converges(partial(seq_abs(a))) {
        converges(partial(seq_abs(a))) = forall(e: Real) {
            e.is_positive implies exists(n0: Nat) {
                cauchy_bound(partial(seq_abs(a)), n0, e)
            }
        }
        forall(eps: Real) {
            if eps.is_positive {
                eps.is_positive implies exists(n0: Nat) {
                    cauchy_bound(partial(seq_abs(a)), n0, eps)
                }
                exists(n0: Nat) {
                    cauchy_bound(partial(seq_abs(a)), n0, eps)
                }
                let n: Nat satisfy {
                    cauchy_bound(partial(seq_abs(a)), n, eps)
                }
                cauchy_bound(partial(seq_abs(a)), n, eps) = forall(i0: Nat, j0: Nat) {
                    n <= i0 and n <= j0 implies partial(seq_abs(a), i0).is_close(partial(seq_abs(a), j0), eps)
                }
                forall(i: Nat, j: Nat) {
                    if n <= i and n <= j {
                        if j <= i {
                            partial_diff_abs_le(a, i, j)
                            (partial(a, i) - partial(a, j)).abs <= partial(seq_abs(a), i) - partial(seq_abs(a), j)
                            abs_gte_zero(partial(a, i) - partial(a, j))
                            (partial(a, i) - partial(a, j)).abs >= Real.0
                            Real.0 <= (partial(a, i) - partial(a, j)).abs
                            lte_trans(Real.0, (partial(a, i) - partial(a, j)).abs, partial(seq_abs(a), i) - partial(seq_abs(a), j))
                            Real.0 <= partial(seq_abs(a), i) - partial(seq_abs(a), j)
                            partial(seq_abs(a), i) - partial(seq_abs(a), j) >= Real.0
                            pos_imp_eq_abs(partial(seq_abs(a), i) - partial(seq_abs(a), j))
                            partial(seq_abs(a), i) - partial(seq_abs(a), j) = (partial(seq_abs(a), i) - partial(seq_abs(a), j)).abs
                            partial(seq_abs(a), i).is_close(partial(seq_abs(a), j), eps)
                            (partial(seq_abs(a), i) - partial(seq_abs(a), j)).abs < eps
                            lte_lt_trans((partial(a, i) - partial(a, j)).abs, partial(seq_abs(a), i) - partial(seq_abs(a), j), eps)
                            (partial(a, i) - partial(a, j)).abs < eps
                            partial(a, i).is_close(partial(a, j), eps)
                        } else {
                            lt_or_lte(i, j)
                            i < j or j <= i
                            not (j <= i)
                            i < j
                            lt_imp_lte(i, j)
                            i <= j
                            partial_diff_abs_le(a, j, i)
                            (partial(a, j) - partial(a, i)).abs <= partial(seq_abs(a), j) - partial(seq_abs(a), i)
                            abs_gte_zero(partial(a, j) - partial(a, i))
                            (partial(a, j) - partial(a, i)).abs >= Real.0
                            Real.0 <= (partial(a, j) - partial(a, i)).abs
                            lte_trans(Real.0, (partial(a, j) - partial(a, i)).abs, partial(seq_abs(a), j) - partial(seq_abs(a), i))
                            Real.0 <= partial(seq_abs(a), j) - partial(seq_abs(a), i)
                            partial(seq_abs(a), j) - partial(seq_abs(a), i) >= Real.0
                            pos_imp_eq_abs(partial(seq_abs(a), j) - partial(seq_abs(a), i))
                            partial(seq_abs(a), j) - partial(seq_abs(a), i) = (partial(seq_abs(a), j) - partial(seq_abs(a), i)).abs
                            partial(seq_abs(a), j).is_close(partial(seq_abs(a), i), eps)
                            (partial(seq_abs(a), j) - partial(seq_abs(a), i)).abs < eps
                            lte_lt_trans((partial(a, j) - partial(a, i)).abs, partial(seq_abs(a), j) - partial(seq_abs(a), i), eps)
                            (partial(a, j) - partial(a, i)).abs < eps
                            partial(a, i) - partial(a, j) = partial(a, i) + -partial(a, j)
                            partial(a, j) - partial(a, i) = partial(a, j) + -partial(a, i)
                            -(partial(a, j) + -partial(a, i)) = -partial(a, j) + partial(a, i)
                            partial(a, i) + -partial(a, j) = -partial(a, j) + partial(a, i)
                            -(partial(a, j) - partial(a, i)) = partial(a, i) + -partial(a, j)
                            partial(a, i) - partial(a, j) = -(partial(a, j) - partial(a, i))
                            abs_neg(partial(a, j) - partial(a, i))
                            (-(partial(a, j) - partial(a, i))).abs = (partial(a, j) - partial(a, i)).abs
                            (partial(a, i) - partial(a, j)).abs = (partial(a, j) - partial(a, i)).abs
                            (partial(a, i) - partial(a, j)).abs < eps
                            partial(a, i).is_close(partial(a, j), eps)
                        }
                    }
                }
                cauchy_bound(partial(a), n, eps)
            }
        }
    }
}

/// If a tail of the series converges, then the series converges.
theorem tail_conv_imp_conv(a: Nat -> Real, n0: Nat) {
    converges(partial(seq_tail(a, n0))) implies converges(partial(a))
} by {
    if converges(partial(seq_tail(a, n0))) {
        converges(partial(seq_tail(a, n0))) = forall(e: Real) {
            e.is_positive implies exists(n1: Nat) {
                cauchy_bound(partial(seq_tail(a, n0)), n1, e)
            }
        }
        forall(eps: Real) {
            if eps.is_positive {
                eps.is_positive implies exists(n1: Nat) {
                    cauchy_bound(partial(seq_tail(a, n0)), n1, eps)
                }
                exists(n1: Nat) {
                    cauchy_bound(partial(seq_tail(a, n0)), n1, eps)
                }
                let m: Nat satisfy {
                    cauchy_bound(partial(seq_tail(a, n0)), m, eps)
                }
                cauchy_bound(partial(seq_tail(a, n0)), m, eps) = forall(i0: Nat, j0: Nat) {
                    m <= i0 and m <= j0 implies partial(seq_tail(a, n0), i0).is_close(partial(seq_tail(a, n0), j0), eps)
                }
                let n: Nat = n0 + m
                forall(i: Nat, j: Nat) {
                    if n <= i and n <= j {
                        let d1: Nat satisfy {
                            n + d1 = i
                        }
                        let d2: Nat satisfy {
                            n + d2 = j
                        }
                        n + d1 = i
                        n + d2 = j
                        n0 + (m + d1) = n + d1
                        n0 + (m + d2) = n + d2
                        partial_tail_decomp(a, n0, m + d1)
                        partial(a, n0 + (m + d1)) = partial(a, n0) + partial(seq_tail(a, n0), m + d1)
                        partial(a, i) = partial(a, n0) + partial(seq_tail(a, n0), m + d1)
                        partial_tail_decomp(a, n0, m + d2)
                        partial(a, j) = partial(a, n0) + partial(seq_tail(a, n0), m + d2)
                        sub_both_eq_sub_add(partial(a, n0) + partial(seq_tail(a, n0), m + d1), partial(a, n0), partial(seq_tail(a, n0), m + d2))
                        (partial(a, n0) + partial(seq_tail(a, n0), m + d1)) - partial(a, n0) - partial(seq_tail(a, n0), m + d2) =
                            (partial(a, n0) + partial(seq_tail(a, n0), m + d1)) - (partial(a, n0) + partial(seq_tail(a, n0), m + d2))
                        (partial(a, n0) + partial(seq_tail(a, n0), m + d1)) - partial(a, n0) = partial(seq_tail(a, n0), m + d1)
                        (partial(a, n0) + partial(seq_tail(a, n0), m + d1)) - partial(a, n0) - partial(seq_tail(a, n0), m + d2) =
                            partial(seq_tail(a, n0), m + d1) - partial(seq_tail(a, n0), m + d2)
                        (partial(a, n0) + partial(seq_tail(a, n0), m + d1)) - (partial(a, n0) + partial(seq_tail(a, n0), m + d2)) =
                            partial(seq_tail(a, n0), m + d1) - partial(seq_tail(a, n0), m + d2)
                        partial(a, i) - partial(a, j) = partial(seq_tail(a, n0), m + d1) - partial(seq_tail(a, n0), m + d2)
                        Nat.0 <= d1
                        lte_add_left(Nat.0, d1, m)
                        m + Nat.0 <= m + d1
                        m + Nat.0 = m
                        m <= m + d1
                        Nat.0 <= d2
                        lte_add_left(Nat.0, d2, m)
                        m + Nat.0 <= m + d2
                        m <= m + d2
                        partial(seq_tail(a, n0), m + d1).is_close(partial(seq_tail(a, n0), m + d2), eps)
                        (partial(seq_tail(a, n0), m + d1) - partial(seq_tail(a, n0), m + d2)).abs < eps
                        (partial(a, i) - partial(a, j)).abs < eps
                        partial(a, i).is_close(partial(a, j), eps)
                    }
                }
                cauchy_bound(partial(a), n, eps)
            }
        }
    }
}

/// Partial sums of nonnegative sequences are increasing.
theorem nonneg_partial_increasing(a: Nat -> Real) {
    is_lower_bound(a, Real.0) implies is_increasing(partial(a))
} by {
    if is_lower_bound(a, Real.0) {
        forall(n: Nat) {
            partial_suc(a, n)
            partial(a, n.suc) = partial(a, n) + a(n)
            is_lower_bound(a, Real.0)
            Real.0 <= a(n)
            lte_refl(partial(a, n))
            partial(a, n) <= partial(a, n)
            add_lte_add(partial(a, n), partial(a, n), Real.0, a(n))
            partial(a, n) + Real.0 <= partial(a, n) + a(n)
            partial(a, n) + Real.0 = partial(a, n)
            partial(a, n) <= partial(a, n) + a(n)
            partial(a, n) <= partial(a, n.suc)
        }
        is_increasing(partial(a))
    }
}

/// The tail of the exponential series converges absolutely when 2|x| <= n0 + 1.
theorem exp_term_tail_abs_series_converges(x: Real, n0: Nat) {
    x.abs * two <= Real.from_rat(Rat.from_nat(n0 + Nat.1))
    implies converges(partial(seq_abs(seq_tail(exp_term(x), n0))))
} by {
    if x.abs * two <= Real.from_rat(Rat.from_nat(n0 + Nat.1)) {
        let c: Real = exp_term(x, n0).abs
        one_half_positive
        Real.0 < Real.one_half
        Real.one_half.is_positive
        pos_imp_eq_abs(Real.one_half)
        Real.one_half = Real.one_half.abs
        Real.one_half.abs = Real.one_half
        one_half_plus_one_half
        Real.one_half + Real.one_half = Real.1
        lt_add_right(Real.0, Real.one_half, Real.one_half)
        Real.0 + Real.one_half < Real.one_half + Real.one_half
        add_zero_left(Real.one_half)
        Real.0 + Real.one_half = Real.one_half
        Real.one_half < Real.1
        Real.one_half.abs < Real.1
        geom_converges(Real.one_half)
        converges(partial(Real.one_half.pow))
        forall(n: Nat) {
            partial_scalar_mul(c, Real.one_half.pow, n)
            c * partial(Real.one_half.pow, n) = partial(mul_fn(c, Real.one_half.pow), n)
            mul_seq(c, partial(Real.one_half.pow), n) = c * partial(Real.one_half.pow, n)
            mul_fn(c, partial(Real.one_half.pow), n) = c * partial(Real.one_half.pow, n)
            mul_fn(c, partial(Real.one_half.pow), n) = mul_seq(c, partial(Real.one_half.pow), n)
        }
        mul_fn(c, partial(Real.one_half.pow)) = mul_seq(c, partial(Real.one_half.pow))
        converges_mul_seq(c, partial(Real.one_half.pow))
        converges(mul_seq(c, partial(Real.one_half.pow)))
        converges(mul_fn(c, partial(Real.one_half.pow)))
        forall(n: Nat) {
            mul_fn(c, partial(Real.one_half.pow), n) = partial(mul_fn(c, Real.one_half.pow), n)
        }
        partial(mul_fn(c, Real.one_half.pow)) = mul_fn(c, partial(Real.one_half.pow))
        converges(partial(mul_fn(c, Real.one_half.pow)))

        forall(j: Nat) {
            exp_term_tail_geom_bound(x, n0, j)
            exp_term(x, n0 + j).abs <= exp_term(x, n0).abs * Real.one_half.pow(j)
            seq_abs(seq_tail(exp_term(x), n0), j) = seq_tail(exp_term(x), n0, j).abs
            seq_tail(exp_term(x), n0, j) = exp_term(x, n0 + j)
            seq_tail(exp_term(x), n0, j).abs = exp_term(x, n0 + j).abs
            mul_fn(c, Real.one_half.pow, j) = c * Real.one_half.pow(j)
            seq_abs(seq_tail(exp_term(x), n0), j) <= mul_fn(c, Real.one_half.pow, j)
        }
        seq_lte(seq_abs(seq_tail(exp_term(x), n0)), mul_fn(c, Real.one_half.pow))

        forall(j: Nat) {
            abs_gte_zero(exp_term(x, n0 + j))
            seq_abs(seq_tail(exp_term(x), n0), j) = seq_tail(exp_term(x), n0, j).abs
            seq_tail(exp_term(x), n0, j) = exp_term(x, n0 + j)
            seq_tail(exp_term(x), n0, j).abs = exp_term(x, n0 + j).abs
            exp_term(x, n0 + j).abs >= Real.0
            seq_abs(seq_tail(exp_term(x), n0), j) >= Real.0
            Real.0 <= seq_abs(seq_tail(exp_term(x), n0), j)
        }
        is_lower_bound(seq_abs(seq_tail(exp_term(x), n0)), Real.0)
        nonneg_partial_increasing(seq_abs(seq_tail(exp_term(x), n0)))
        is_increasing(partial(seq_abs(seq_tail(exp_term(x), n0))))

        forall(j: Nat) {
            abs_gte_zero(exp_term(x, n0))
            c >= Real.0
            one_half_positive
            Real.0 < Real.one_half
            lt_imp_lte(Real.0, Real.one_half)
            Real.one_half >= Real.0
            real_pow_nonneg(Real.one_half, j)
            Real.one_half.pow(j) >= Real.0
            mul_nonneg(c, Real.one_half.pow(j))
            c * Real.one_half.pow(j) >= Real.0
            mul_fn(c, Real.one_half.pow, j) = c * Real.one_half.pow(j)
            mul_fn(c, Real.one_half.pow, j) >= Real.0
            Real.0 <= mul_fn(c, Real.one_half.pow, j)
        }
        is_lower_bound(mul_fn(c, Real.one_half.pow), Real.0)
        nonneg_partial_increasing(mul_fn(c, Real.one_half.pow))
        is_increasing(partial(mul_fn(c, Real.one_half.pow)))

        partial_seq_lte(seq_abs(seq_tail(exp_term(x), n0)), mul_fn(c, Real.one_half.pow))
        seq_lte(partial(seq_abs(seq_tail(exp_term(x), n0))), partial(mul_fn(c, Real.one_half.pow)))
        increasing_convergent_bounded_by_limit(partial(mul_fn(c, Real.one_half.pow)))
        is_upper_bound(partial(mul_fn(c, Real.one_half.pow)), limit(partial(mul_fn(c, Real.one_half.pow))))
        forall(n: Nat) {
            seq_lte(partial(seq_abs(seq_tail(exp_term(x), n0))), partial(mul_fn(c, Real.one_half.pow)))
            partial(seq_abs(seq_tail(exp_term(x), n0)), n) <= partial(mul_fn(c, Real.one_half.pow), n)
            is_upper_bound(partial(mul_fn(c, Real.one_half.pow)), limit(partial(mul_fn(c, Real.one_half.pow))))
            partial(mul_fn(c, Real.one_half.pow), n) <= limit(partial(mul_fn(c, Real.one_half.pow)))
            lte_trans(partial(seq_abs(seq_tail(exp_term(x), n0)), n), partial(mul_fn(c, Real.one_half.pow), n), limit(partial(mul_fn(c, Real.one_half.pow))))
            partial(seq_abs(seq_tail(exp_term(x), n0)), n) <= limit(partial(mul_fn(c, Real.one_half.pow)))
        }
        is_upper_bound(partial(seq_abs(seq_tail(exp_term(x), n0))), limit(partial(mul_fn(c, Real.one_half.pow))))
        monotone_convergence_principle(partial(seq_abs(seq_tail(exp_term(x), n0))), limit(partial(mul_fn(c, Real.one_half.pow))))
        converges(partial(seq_abs(seq_tail(exp_term(x), n0))))
    }
}

/// The real exponential series converges absolutely for every x.
theorem exp_term_abs_series_converges(x: Real) {
    converges(partial(seq_abs(exp_term(x))))
} by {
    exists_ratio_index(x)
    let n0: Nat satisfy {
        x.abs * two <= Real.from_rat(Rat.from_nat(n0 + Nat.1))
    }
    exp_term_tail_abs_series_converges(x, n0)
    converges(partial(seq_abs(seq_tail(exp_term(x), n0))))
    seq_abs_tail_comm(exp_term(x), n0)
    seq_abs(seq_tail(exp_term(x), n0)) = seq_tail(seq_abs(exp_term(x)), n0)
    converges(partial(seq_tail(seq_abs(exp_term(x)), n0)))
    tail_conv_imp_conv(seq_abs(exp_term(x)), n0)
    converges(partial(seq_abs(exp_term(x))))
}

/// The real exponential series converges for every x.
theorem exp_term_series_converges(x: Real) {
    converges(partial(exp_term(x)))
} by {
    exp_term_abs_series_converges(x)
    abs_series_imp_series(exp_term(x))
    converges(partial(exp_term(x)))
}

// ---------------------------------------------------------------------------
// Convergence of the complex exponential series, and the real-embedding
// consistency of the complex exponential.
// ---------------------------------------------------------------------------

/// The absolute value of the real part of a complex exponential term is at most
/// the real exponential term at the modulus.
theorem complex_exp_term_re_abs_le(z: Complex, n: Nat) {
    seq_abs(re_seq(complex_exp_term(z)))(n) <= exp_term(z.modulus, n)
} by {
    seq_abs(re_seq(complex_exp_term(z)))(n) = re_seq(complex_exp_term(z), n).abs
    re_seq(complex_exp_term(z), n) = complex_exp_term(z, n).re
    re_abs_le_modulus(complex_exp_term(z, n))
    complex_exp_term(z, n).re.abs <= complex_exp_term(z, n).modulus
    complex_exp_term_modulus(z, n)
    complex_exp_term(z, n).modulus = exp_term(z.modulus, n)
    seq_abs(re_seq(complex_exp_term(z)))(n) <= exp_term(z.modulus, n)
}

/// The absolute value of the imaginary part of a complex exponential term is at
/// most the real exponential term at the modulus.
theorem complex_exp_term_im_abs_le(z: Complex, n: Nat) {
    seq_abs(im_seq(complex_exp_term(z)))(n) <= exp_term(z.modulus, n)
} by {
    seq_abs(im_seq(complex_exp_term(z)))(n) = im_seq(complex_exp_term(z), n).abs
    im_seq(complex_exp_term(z), n) = complex_exp_term(z, n).im
    im_abs_le_modulus(complex_exp_term(z, n))
    complex_exp_term(z, n).im.abs <= complex_exp_term(z, n).modulus
    complex_exp_term_modulus(z, n)
    complex_exp_term(z, n).modulus = exp_term(z.modulus, n)
    seq_abs(im_seq(complex_exp_term(z)))(n) <= exp_term(z.modulus, n)
}

/// The real-part sequence of the complex exponential series is absolutely convergent.
theorem complex_exp_series_re_abs_conv(z: Complex) {
    converges(partial(seq_abs(re_seq(complex_exp_term(z)))))
} by {
    forall(n: Nat) {
        abs_gte_zero(re_seq(complex_exp_term(z), n))
        re_seq(complex_exp_term(z), n).abs >= Real.0
        seq_abs(re_seq(complex_exp_term(z)), n) = re_seq(complex_exp_term(z), n).abs
        seq_abs(re_seq(complex_exp_term(z)), n) >= Real.0
        Real.0 <= seq_abs(re_seq(complex_exp_term(z)), n)
    }
    is_lower_bound(seq_abs(re_seq(complex_exp_term(z))), Real.0)
    nonneg_partial_increasing(seq_abs(re_seq(complex_exp_term(z))))
    is_increasing(partial(seq_abs(re_seq(complex_exp_term(z)))))

    forall(n: Nat) {
        complex_exp_term_re_abs_le(z, n)
        seq_abs(re_seq(complex_exp_term(z)))(n) <= exp_term(z.modulus, n)
        lte_abs(exp_term(z.modulus, n))
        exp_term(z.modulus, n) <= exp_term(z.modulus, n).abs
        seq_abs(exp_term(z.modulus), n) = exp_term(z.modulus, n).abs
        exp_term(z.modulus, n) <= seq_abs(exp_term(z.modulus), n)
        lte_trans(seq_abs(re_seq(complex_exp_term(z)))(n), exp_term(z.modulus, n), seq_abs(exp_term(z.modulus), n))
        seq_abs(re_seq(complex_exp_term(z)))(n) <= seq_abs(exp_term(z.modulus), n)
    }
    seq_lte(seq_abs(re_seq(complex_exp_term(z))), seq_abs(exp_term(z.modulus)))
    partial_seq_lte(seq_abs(re_seq(complex_exp_term(z))), seq_abs(exp_term(z.modulus)))
    seq_lte(partial(seq_abs(re_seq(complex_exp_term(z)))), partial(seq_abs(exp_term(z.modulus))))

    forall(n: Nat) {
        abs_gte_zero(exp_term(z.modulus, n))
        exp_term(z.modulus, n).abs >= Real.0
        seq_abs(exp_term(z.modulus), n) = exp_term(z.modulus, n).abs
        seq_abs(exp_term(z.modulus), n) >= Real.0
        Real.0 <= seq_abs(exp_term(z.modulus), n)
    }
    is_lower_bound(seq_abs(exp_term(z.modulus)), Real.0)
    nonneg_partial_increasing(seq_abs(exp_term(z.modulus)))
    is_increasing(partial(seq_abs(exp_term(z.modulus))))
    forall(n: Nat) {
        abs_gte_zero(exp_term(z.modulus, n))
        exp_term(z.modulus, n).abs >= Real.0
        seq_abs(exp_term(z.modulus), n) = exp_term(z.modulus, n).abs
        seq_abs(exp_term(z.modulus), n) >= Real.0
        Real.0 <= seq_abs(exp_term(z.modulus), n)
    }
    is_lower_bound(seq_abs(exp_term(z.modulus)), Real.0)
    nonneg_partial_increasing(seq_abs(exp_term(z.modulus)))
    is_increasing(partial(seq_abs(exp_term(z.modulus))))
    exp_term_abs_series_converges(z.modulus)
    converges(partial(seq_abs(exp_term(z.modulus))))
    increasing_convergent_bounded_by_limit(partial(seq_abs(exp_term(z.modulus))))
    is_upper_bound(partial(seq_abs(exp_term(z.modulus))), limit(partial(seq_abs(exp_term(z.modulus)))))
    forall(n: Nat) {
        seq_lte(partial(seq_abs(re_seq(complex_exp_term(z)))), partial(seq_abs(exp_term(z.modulus))))
        partial(seq_abs(re_seq(complex_exp_term(z))), n) <= partial(seq_abs(exp_term(z.modulus)), n)
        is_upper_bound(partial(seq_abs(exp_term(z.modulus))), limit(partial(seq_abs(exp_term(z.modulus)))))
        partial(seq_abs(exp_term(z.modulus)), n) <= limit(partial(seq_abs(exp_term(z.modulus))))
        lte_trans(partial(seq_abs(re_seq(complex_exp_term(z))), n), partial(seq_abs(exp_term(z.modulus)), n), limit(partial(seq_abs(exp_term(z.modulus)))))
        partial(seq_abs(re_seq(complex_exp_term(z))), n) <= limit(partial(seq_abs(exp_term(z.modulus))))
    }
    is_upper_bound(partial(seq_abs(re_seq(complex_exp_term(z)))), limit(partial(seq_abs(exp_term(z.modulus)))))
    monotone_convergence_principle(partial(seq_abs(re_seq(complex_exp_term(z)))), limit(partial(seq_abs(exp_term(z.modulus)))))
    converges(partial(seq_abs(re_seq(complex_exp_term(z)))))
}

/// The imaginary-part sequence of the complex exponential series is absolutely convergent.
theorem complex_exp_series_im_abs_conv(z: Complex) {
    converges(partial(seq_abs(im_seq(complex_exp_term(z)))))
} by {
    forall(n: Nat) {
        abs_gte_zero(im_seq(complex_exp_term(z), n))
        im_seq(complex_exp_term(z), n).abs >= Real.0
        seq_abs(im_seq(complex_exp_term(z)), n) = im_seq(complex_exp_term(z), n).abs
        seq_abs(im_seq(complex_exp_term(z)), n) >= Real.0
        Real.0 <= seq_abs(im_seq(complex_exp_term(z)), n)
    }
    is_lower_bound(seq_abs(im_seq(complex_exp_term(z))), Real.0)
    nonneg_partial_increasing(seq_abs(im_seq(complex_exp_term(z))))
    is_increasing(partial(seq_abs(im_seq(complex_exp_term(z)))))

    forall(n: Nat) {
        complex_exp_term_im_abs_le(z, n)
        seq_abs(im_seq(complex_exp_term(z)))(n) <= exp_term(z.modulus, n)
        lte_abs(exp_term(z.modulus, n))
        exp_term(z.modulus, n) <= exp_term(z.modulus, n).abs
        seq_abs(exp_term(z.modulus), n) = exp_term(z.modulus, n).abs
        exp_term(z.modulus, n) <= seq_abs(exp_term(z.modulus), n)
        lte_trans(seq_abs(im_seq(complex_exp_term(z)))(n), exp_term(z.modulus, n), seq_abs(exp_term(z.modulus), n))
        seq_abs(im_seq(complex_exp_term(z)))(n) <= seq_abs(exp_term(z.modulus), n)
    }
    seq_lte(seq_abs(im_seq(complex_exp_term(z))), seq_abs(exp_term(z.modulus)))
    partial_seq_lte(seq_abs(im_seq(complex_exp_term(z))), seq_abs(exp_term(z.modulus)))
    seq_lte(partial(seq_abs(im_seq(complex_exp_term(z)))), partial(seq_abs(exp_term(z.modulus))))

    forall(n: Nat) {
        abs_gte_zero(exp_term(z.modulus, n))
        exp_term(z.modulus, n).abs >= Real.0
        seq_abs(exp_term(z.modulus), n) = exp_term(z.modulus, n).abs
        seq_abs(exp_term(z.modulus), n) >= Real.0
        Real.0 <= seq_abs(exp_term(z.modulus), n)
    }
    is_lower_bound(seq_abs(exp_term(z.modulus)), Real.0)
    nonneg_partial_increasing(seq_abs(exp_term(z.modulus)))
    is_increasing(partial(seq_abs(exp_term(z.modulus))))

    exp_term_abs_series_converges(z.modulus)
    converges(partial(seq_abs(exp_term(z.modulus))))
    increasing_convergent_bounded_by_limit(partial(seq_abs(exp_term(z.modulus))))
    is_upper_bound(partial(seq_abs(exp_term(z.modulus))), limit(partial(seq_abs(exp_term(z.modulus)))))
    forall(n: Nat) {
        seq_lte(partial(seq_abs(im_seq(complex_exp_term(z)))), partial(seq_abs(exp_term(z.modulus))))
        partial(seq_abs(im_seq(complex_exp_term(z))), n) <= partial(seq_abs(exp_term(z.modulus)), n)
        is_upper_bound(partial(seq_abs(exp_term(z.modulus))), limit(partial(seq_abs(exp_term(z.modulus)))))
        partial(seq_abs(exp_term(z.modulus)), n) <= limit(partial(seq_abs(exp_term(z.modulus))))
        lte_trans(partial(seq_abs(im_seq(complex_exp_term(z))), n), partial(seq_abs(exp_term(z.modulus)), n), limit(partial(seq_abs(exp_term(z.modulus)))))
        partial(seq_abs(im_seq(complex_exp_term(z))), n) <= limit(partial(seq_abs(exp_term(z.modulus))))
    }
    is_upper_bound(partial(seq_abs(im_seq(complex_exp_term(z)))), limit(partial(seq_abs(exp_term(z.modulus)))))
    monotone_convergence_principle(partial(seq_abs(im_seq(complex_exp_term(z)))), limit(partial(seq_abs(exp_term(z.modulus)))))
    converges(partial(seq_abs(im_seq(complex_exp_term(z)))))
}

/// The real-part sequence of the partial sums of the complex exponential series converges.
theorem complex_exp_series_re_converges(z: Complex) {
    converges(partial(re_seq(complex_exp_term(z))))
} by {
    complex_exp_series_re_abs_conv(z)
    abs_series_imp_series(re_seq(complex_exp_term(z)))
    converges(partial(re_seq(complex_exp_term(z))))
}

/// The imaginary-part sequence of the partial sums of the complex exponential series converges.
theorem complex_exp_series_im_converges(z: Complex) {
    converges(partial(im_seq(complex_exp_term(z))))
} by {
    complex_exp_series_im_abs_conv(z)
    abs_series_imp_series(im_seq(complex_exp_term(z)))
    converges(partial(im_seq(complex_exp_term(z))))
}

/// The partial sums of the complex exponential series converge componentwise.
theorem complex_exp_series_converges_to(z: Complex) {
    complex_converges_to(partial(complex_exp_term(z)),
        Complex.new(limit(partial(re_seq(complex_exp_term(z)))), limit(partial(im_seq(complex_exp_term(z))))))
} by {
    complex_exp_series_re_converges(z)
    complex_exp_series_im_converges(z)
    converges(partial(re_seq(complex_exp_term(z))))
    converges(partial(im_seq(complex_exp_term(z))))
    converges_imp_converges_to(partial(re_seq(complex_exp_term(z))))
    converges_imp_converges_to(partial(im_seq(complex_exp_term(z))))
    converges_to(partial(re_seq(complex_exp_term(z))), limit(partial(re_seq(complex_exp_term(z)))))
    converges_to(partial(im_seq(complex_exp_term(z))), limit(partial(im_seq(complex_exp_term(z)))))
    forall(n: Nat) {
        partial_re_seq(complex_exp_term(z), n)
    }
    forall(n: Nat) {
        partial_im_seq(complex_exp_term(z), n)
    }
    re_seq(partial(complex_exp_term(z))) = partial(re_seq(complex_exp_term(z)))
    im_seq(partial(complex_exp_term(z))) = partial(im_seq(complex_exp_term(z)))
    converges_to(re_seq(partial(complex_exp_term(z))), limit(partial(re_seq(complex_exp_term(z)))))
    converges_to(im_seq(partial(complex_exp_term(z))), limit(partial(im_seq(complex_exp_term(z)))))
    componentwise_imp_complex_converges_to(partial(complex_exp_term(z)),
        Complex.new(limit(partial(re_seq(complex_exp_term(z)))), limit(partial(im_seq(complex_exp_term(z))))))
}

/// The partial sums of the complex exponential series converge for every complex number.
theorem complex_exp_series_converges(z: Complex) {
    complex_converges(partial(complex_exp_term(z)))
} by {
    complex_exp_series_converges_to(z)
    complex_converges_to_imp_converges(partial(complex_exp_term(z)),
        Complex.new(limit(partial(re_seq(complex_exp_term(z)))), limit(partial(im_seq(complex_exp_term(z))))))
    complex_converges(partial(complex_exp_term(z)))
}

/// The complex exponential of an embedded real number is the embedding of the real
/// exponential: (from_real(x)).exp = from_real(x.exp).
theorem complex_exp_term_from_real(x: Real, n: Nat) {
    complex_exp_term(Complex.from_real(x), n) = Complex.from_real(exp_term(x, n))
} by {
    ring_hom_pow(complex_from_real_ring_hom, x, n)
    complex_from_real_ring_hom_hom
    Complex.from_real(x.pow(n)) = Complex.from_real(x).pow(n)
    complex_exp_term(Complex.from_real(x), n) = Complex.from_real(x).pow(n) / Complex.from_real(Real.from_rat(Rat.from_nat(n.factorial)))
    Complex.from_real(x).pow(n) = Complex.from_real(x.pow(n))
    complex_exp_term(Complex.from_real(x), n) = Complex.from_real(x.pow(n)) / Complex.from_real(Real.from_rat(Rat.from_nat(n.factorial)))
    div_from_real(x.pow(n), Real.from_rat(Rat.from_nat(n.factorial)))
    Complex.from_real(x.pow(n)) / Complex.from_real(Real.from_rat(Rat.from_nat(n.factorial))) =
        Complex.from_real(x.pow(n) / Real.from_rat(Rat.from_nat(n.factorial)))
    complex_exp_term(Complex.from_real(x), n) = Complex.from_real(x.pow(n) / Real.from_rat(Rat.from_nat(n.factorial)))
    exp_term(x, n) = x.pow(n) / Real.from_rat(Rat.from_nat(n.factorial))
    complex_exp_term(Complex.from_real(x), n) = Complex.from_real(exp_term(x, n))
}

/// The real parts of the complex exponential terms at an embedded real number are
/// the real exponential terms.
theorem complex_exp_term_from_real_re(x: Real, n: Nat) {
    re_seq(complex_exp_term(Complex.from_real(x)), n) = exp_term(x, n)
} by {
    complex_exp_term_from_real(x, n)
    complex_exp_term(Complex.from_real(x), n) = Complex.from_real(exp_term(x, n))
    re_seq(complex_exp_term(Complex.from_real(x)), n) = complex_exp_term(Complex.from_real(x), n).re
    complex_exp_term(Complex.from_real(x), n).re = Complex.from_real(exp_term(x, n)).re
    re_from_real(exp_term(x, n))
    Complex.from_real(exp_term(x, n)).re = exp_term(x, n)
    re_seq(complex_exp_term(Complex.from_real(x)), n) = exp_term(x, n)
}

/// The imaginary parts of the complex exponential terms at an embedded real number vanish.
theorem complex_exp_term_from_real_im(x: Real, n: Nat) {
    im_seq(complex_exp_term(Complex.from_real(x)), n) = Real.0
} by {
    complex_exp_term_from_real(x, n)
    complex_exp_term(Complex.from_real(x), n) = Complex.from_real(exp_term(x, n))
    im_seq(complex_exp_term(Complex.from_real(x)), n) = complex_exp_term(Complex.from_real(x), n).im
    complex_exp_term(Complex.from_real(x), n).im = Complex.from_real(exp_term(x, n)).im
    im_from_real(exp_term(x, n))
    Complex.from_real(exp_term(x, n)).im = Real.0
    im_seq(complex_exp_term(Complex.from_real(x)), n) = Real.0
}

/// The partial sums of the real parts of the complex exponential series at an embedded
/// real number are the partial sums of the real exponential series.
theorem complex_exp_from_real_partial_re(x: Real, n: Nat) {
    partial(re_seq(complex_exp_term(Complex.from_real(x))), n) = partial(exp_term(x), n)
} by {
    forall(k: Nat) {
        if k < n {
            complex_exp_term_from_real_re(x, k)
            re_seq(complex_exp_term(Complex.from_real(x)), k) = exp_term(x, k)
        }
    }
    partial_pointwise_eq(re_seq(complex_exp_term(Complex.from_real(x))), exp_term(x), n)
    partial(re_seq(complex_exp_term(Complex.from_real(x))), n) = partial(exp_term(x), n)
}

/// The complex exponential agrees with the embedding of the real exponential on
/// embedded real numbers.
theorem complex_exp_from_real(x: Real) {
    complex_exp(Complex.from_real(x)) = Complex.from_real(x.exp)
} by {
    forall(n: Nat) {
        complex_exp_from_real_partial_re(x, n)
        partial(re_seq(complex_exp_term(Complex.from_real(x))), n) = partial(exp_term(x), n)
    }
    partial(re_seq(complex_exp_term(Complex.from_real(x)))) = partial(exp_term(x))
    exp_term_series_converges(x)
    converges(partial(exp_term(x)))
    converges_imp_converges_to(partial(exp_term(x)))
    converges_to(partial(exp_term(x)), limit(partial(exp_term(x))))
    x.exp = limit(partial(exp_term(x)))
    converges_to(partial(exp_term(x)), x.exp)
    converges_to(partial(re_seq(complex_exp_term(Complex.from_real(x)))), x.exp)
    forall(n: Nat) {
        partial_re_seq(complex_exp_term(Complex.from_real(x)), n)
    }
    re_seq(partial(complex_exp_term(Complex.from_real(x)))) = partial(re_seq(complex_exp_term(Complex.from_real(x))))
    converges_to(re_seq(partial(complex_exp_term(Complex.from_real(x)))), x.exp)

    forall(n: Nat) {
        complex_exp_term_from_real_im(x, n)
        im_seq(complex_exp_term(Complex.from_real(x)), n) = Real.0
        im_seq(complex_exp_term(Complex.from_real(x)), n) = constant[Nat, Real](Real.0, n)
    }
    im_seq(complex_exp_term(Complex.from_real(x))) = constant[Nat, Real](Real.0)
    forall(n: Nat) {
        partial(im_seq(complex_exp_term(Complex.from_real(x))), n) = partial(constant[Nat, Real](Real.0), n)
        partial_zero_seq_const(n)
        partial(constant[Nat, Real](Real.0), n) = Real.0
        partial(im_seq(complex_exp_term(Complex.from_real(x))), n) = Real.0
        partial(im_seq(complex_exp_term(Complex.from_real(x))), n) = constant[Nat, Real](Real.0, n)
    }
    partial(im_seq(complex_exp_term(Complex.from_real(x)))) = constant[Nat, Real](Real.0)
    const_converges_to(Real.0)
    converges_to(constant[Nat, Real](Real.0), Real.0)
    converges_to(partial(im_seq(complex_exp_term(Complex.from_real(x)))), Real.0)
    forall(n: Nat) {
        partial_im_seq(complex_exp_term(Complex.from_real(x)), n)
    }
    im_seq(partial(complex_exp_term(Complex.from_real(x)))) = partial(im_seq(complex_exp_term(Complex.from_real(x))))
    converges_to(im_seq(partial(complex_exp_term(Complex.from_real(x)))), Real.0)

    componentwise_imp_complex_converges_to(partial(complex_exp_term(Complex.from_real(x))), Complex.from_real(x.exp))
    complex_converges_to(partial(complex_exp_term(Complex.from_real(x))), Complex.from_real(x.exp))
    complex_converges_to_imp_limit(partial(complex_exp_term(Complex.from_real(x))), Complex.from_real(x.exp))
    complex_limit(partial(complex_exp_term(Complex.from_real(x)))) = Complex.from_real(x.exp)
    complex_exp(Complex.from_real(x)) = complex_limit(partial(complex_exp_term(Complex.from_real(x))))
    complex_exp(Complex.from_real(x)) = Complex.from_real(x.exp)
}

// ---------------------------------------------------------------------------
// The Cauchy product of complex series, used to prove the functional equation
// (z + w).exp = z.exp * w.exp.  The complex Cauchy product converges
// componentwise, so the real Cauchy product machinery of src/real/cauchy.ac
// applies to the real and imaginary parts separately.
// ---------------------------------------------------------------------------

/// The coefficient function for the complex Cauchy product at index n.
/// For a fixed n, this returns a function mapping k to a(k) * b(n-k).
define complex_cauchy_coefficient(a: Nat -> Complex, b: Nat -> Complex, n: Nat) -> (Nat -> Complex) {
    function(k: Nat) { a(k) * b(n - k) }
}

/// The complex Cauchy product of two sequences at index n.
/// This computes ∑_{k=0}^{n} a(k) * b(n-k).
define complex_cauchy_product(a: Nat -> Complex, b: Nat -> Complex, n: Nat) -> Complex {
    partial(complex_cauchy_coefficient(a, b, n), n.suc)
}

/// The sequence of complex Cauchy products.
define complex_cauchy_seq(a: Nat -> Complex, b: Nat -> Complex) -> (Nat -> Complex) {
    function(n: Nat) { complex_cauchy_product(a, b, n) }
}

/// The pointwise difference of two real sequences.
define sub_pointwise(p: Nat -> Real, q: Nat -> Real, k: Nat) -> Real {
    p(k) - q(k)
}

/// The partial sum of a pointwise difference is the difference of the partial sums.
theorem partial_sub_pointwise(p: Nat -> Real, q: Nat -> Real, n: Nat) {
    partial(sub_pointwise(p, q), n) = partial(p, n) - partial(q, n)
} by {
    define h(m: Nat) -> Bool {
        partial(sub_pointwise(p, q), m) = partial(p, m) - partial(q, m)
    }
    partial(sub_pointwise(p, q), Nat.0) = Real.0
    partial(p, Nat.0) = Real.0
    partial(q, Nat.0) = Real.0
    partial(p, Nat.0) - partial(q, Nat.0) = Real.0
    h(Nat.0)
    forall(m: Nat) {
        if h(m) {
            partial(sub_pointwise(p, q), m) = partial(p, m) - partial(q, m)
            partial_suc(sub_pointwise(p, q), m)
            partial(sub_pointwise(p, q), m.suc) = partial(sub_pointwise(p, q), m) + sub_pointwise(p, q, m)
            sub_pointwise(p, q, m) = p(m) - q(m)
            partial_suc(p, m)
            partial(p, m.suc) = partial(p, m) + p(m)
            partial_suc(q, m)
            partial(q, m.suc) = partial(q, m) + q(m)
            partial(p, m.suc) - partial(q, m.suc) = (partial(p, m) + p(m)) - (partial(q, m) + q(m))
            (partial(p, m) + p(m)) - (partial(q, m) + q(m)) = (partial(p, m) + p(m)) + (-(partial(q, m) + q(m)))
            neg_distrib(partial(q, m), q(m))
            -(partial(q, m) + q(m)) = -partial(q, m) + -q(m)
            (partial(p, m) + p(m)) + (-(partial(q, m) + q(m))) = (partial(p, m) + p(m)) + (-partial(q, m) + -q(m))
            add_assoc(partial(p, m) + p(m), -partial(q, m), -q(m))
            (partial(p, m) + p(m)) + (-partial(q, m) + -q(m)) = ((partial(p, m) + p(m)) + -partial(q, m)) + -q(m)
            add_assoc(partial(p, m), p(m), -partial(q, m))
            (partial(p, m) + p(m)) + -partial(q, m) = partial(p, m) + (p(m) + -partial(q, m))
            add_comm(p(m), -partial(q, m))
            p(m) + -partial(q, m) = -partial(q, m) + p(m)
            partial(p, m) + (p(m) + -partial(q, m)) = partial(p, m) + (-partial(q, m) + p(m))
            add_assoc(partial(p, m), -partial(q, m), p(m))
            (partial(p, m) + -partial(q, m)) + p(m) = partial(p, m) + (-partial(q, m) + p(m))
            ((partial(p, m) + p(m)) + -partial(q, m)) + -q(m) = (partial(p, m) + -partial(q, m)) + p(m) + -q(m)
            add_assoc(partial(p, m) + -partial(q, m), p(m), -q(m))
            (partial(p, m) + -partial(q, m)) + (p(m) + -q(m)) = ((partial(p, m) + -partial(q, m)) + p(m)) + -q(m)
            (partial(p, m) + p(m)) + (-partial(q, m) + -q(m)) = (partial(p, m) + -partial(q, m)) + (p(m) + -q(m))
            (partial(p, m) + -partial(q, m)) + (p(m) + -q(m)) = (partial(p, m) - partial(q, m)) + (p(m) - q(m))
            partial(sub_pointwise(p, q), m.suc) = (partial(p, m) - partial(q, m)) + (p(m) - q(m))
            partial(sub_pointwise(p, q), m.suc) = partial(p, m.suc) - partial(q, m.suc)
            h(m.suc)
        }
    }
    h(Nat.0) and forall(m: Nat) { h(m) implies h(m.suc) }
    alt_induction(h)
    h(n)
}

/// A sequence pointwise equal to a difference inherits convergence to the
/// difference of the limits.
theorem sub_pointwise_converges_to(a: Nat -> Real, b: Nat -> Real, c: Nat -> Real, x: Real, y: Real) {
    converges_to(a, x) and converges_to(b, y) and forall(n: Nat) { c(n) = a(n) - b(n) }
    implies converges_to(c, x - y)
} by {
    if converges_to(a, x) and converges_to(b, y) and forall(n: Nat) { c(n) = a(n) - b(n) } {
        define d(k: Nat) -> Real {
            -b(k)
        }
        forall(n: Nat) {
            d(n) = -b(n)
        }
        neg_pointwise_converges_to(b, d, y)
        converges_to(d, -y)
        forall(n: Nat) {
            c(n) = a(n) - b(n)
            a(n) - b(n) = a(n) + (-b(n))
            d(n) = -b(n)
            c(n) = a(n) + d(n)
        }
        add_pointwise_converges_to(a, d, c, x, -y)
        converges_to(c, x + (-y))
        x + (-y) = x - y
        converges_to(c, x - y)
    }
}

/// The real part of the k-th coefficient of the complex Cauchy product is the
/// difference of the real Cauchy coefficients of the real and imaginary parts.
theorem complex_cauchy_coefficient_re(a: Nat -> Complex, b: Nat -> Complex, n: Nat, k: Nat) {
    re_seq(complex_cauchy_coefficient(a, b, n), k) =
        cauchy_coefficient(re_seq(a), re_seq(b), n)(k) - cauchy_coefficient(im_seq(a), im_seq(b), n)(k)
} by {
    re_seq(complex_cauchy_coefficient(a, b, n), k) = complex_cauchy_coefficient(a, b, n, k).re
    complex_cauchy_coefficient(a, b, n, k) = a(k) * b(n - k)
    complex_cauchy_coefficient(a, b, n, k).re = (a(k) * b(n - k)).re
    re_mul(a(k), b(n - k))
    (a(k) * b(n - k)).re = a(k).re * b(n - k).re - a(k).im * b(n - k).im
    re_seq(a, k) = a(k).re
    re_seq(b, n - k) = b(n - k).re
    im_seq(a, k) = a(k).im
    im_seq(b, n - k) = b(n - k).im
    a(k).re * b(n - k).re - a(k).im * b(n - k).im =
        re_seq(a, k) * re_seq(b, n - k) - im_seq(a, k) * im_seq(b, n - k)
    cauchy_coefficient(re_seq(a), re_seq(b), n)(k) = re_seq(a, k) * re_seq(b, n - k)
    cauchy_coefficient(im_seq(a), im_seq(b), n)(k) = im_seq(a, k) * im_seq(b, n - k)
    re_seq(complex_cauchy_coefficient(a, b, n), k) =
        cauchy_coefficient(re_seq(a), re_seq(b), n)(k) - cauchy_coefficient(im_seq(a), im_seq(b), n)(k)
}

/// The imaginary part of the k-th coefficient of the complex Cauchy product is
/// the sum of the real Cauchy coefficients of the mixed real and imaginary parts.
theorem complex_cauchy_coefficient_im(a: Nat -> Complex, b: Nat -> Complex, n: Nat, k: Nat) {
    im_seq(complex_cauchy_coefficient(a, b, n), k) =
        cauchy_coefficient(re_seq(a), im_seq(b), n)(k) + cauchy_coefficient(im_seq(a), re_seq(b), n)(k)
} by {
    im_seq(complex_cauchy_coefficient(a, b, n), k) = complex_cauchy_coefficient(a, b, n, k).im
    complex_cauchy_coefficient(a, b, n, k) = a(k) * b(n - k)
    complex_cauchy_coefficient(a, b, n, k).im = (a(k) * b(n - k)).im
    im_mul(a(k), b(n - k))
    (a(k) * b(n - k)).im = a(k).re * b(n - k).im + a(k).im * b(n - k).re
    re_seq(a, k) = a(k).re
    im_seq(b, n - k) = b(n - k).im
    im_seq(a, k) = a(k).im
    re_seq(b, n - k) = b(n - k).re
    a(k).re * b(n - k).im + a(k).im * b(n - k).re =
        re_seq(a, k) * im_seq(b, n - k) + im_seq(a, k) * re_seq(b, n - k)
    cauchy_coefficient(re_seq(a), im_seq(b), n)(k) = re_seq(a, k) * im_seq(b, n - k)
    cauchy_coefficient(im_seq(a), re_seq(b), n)(k) = im_seq(a, k) * re_seq(b, n - k)
    im_seq(complex_cauchy_coefficient(a, b, n), k) =
        cauchy_coefficient(re_seq(a), im_seq(b), n)(k) + cauchy_coefficient(im_seq(a), re_seq(b), n)(k)
}

/// The real part of the complex Cauchy product at index n is the difference of
/// the real Cauchy products of the real and imaginary parts.
theorem complex_cauchy_product_re(a: Nat -> Complex, b: Nat -> Complex, n: Nat) {
    re_seq(complex_cauchy_seq(a, b), n) =
        cauchy_product(re_seq(a), re_seq(b), n) - cauchy_product(im_seq(a), im_seq(b), n)
} by {
    re_seq(complex_cauchy_seq(a, b), n) = complex_cauchy_seq(a, b, n).re
    complex_cauchy_seq(a, b, n) = complex_cauchy_product(a, b, n)
    complex_cauchy_seq(a, b, n).re = complex_cauchy_product(a, b, n).re
    complex_cauchy_product(a, b, n) = partial(complex_cauchy_coefficient(a, b, n), n.suc)
    complex_cauchy_product(a, b, n).re = partial(complex_cauchy_coefficient(a, b, n), n.suc).re
    partial_re_seq(complex_cauchy_coefficient(a, b, n), n.suc)
    partial(complex_cauchy_coefficient(a, b, n), n.suc).re =
        partial(re_seq(complex_cauchy_coefficient(a, b, n)), n.suc)
    forall(k: Nat) {
        if k < n.suc {
            complex_cauchy_coefficient_re(a, b, n, k)
            re_seq(complex_cauchy_coefficient(a, b, n), k) =
                cauchy_coefficient(re_seq(a), re_seq(b), n)(k) - cauchy_coefficient(im_seq(a), im_seq(b), n)(k)
            sub_pointwise(cauchy_coefficient(re_seq(a), re_seq(b), n), cauchy_coefficient(im_seq(a), im_seq(b), n), k) =
                cauchy_coefficient(re_seq(a), re_seq(b), n)(k) - cauchy_coefficient(im_seq(a), im_seq(b), n)(k)
            re_seq(complex_cauchy_coefficient(a, b, n), k) =
                sub_pointwise(cauchy_coefficient(re_seq(a), re_seq(b), n), cauchy_coefficient(im_seq(a), im_seq(b), n), k)
        }
    }
    partial_pointwise_eq(re_seq(complex_cauchy_coefficient(a, b, n)),
        sub_pointwise(cauchy_coefficient(re_seq(a), re_seq(b), n), cauchy_coefficient(im_seq(a), im_seq(b), n)), n.suc)
    partial(re_seq(complex_cauchy_coefficient(a, b, n)), n.suc) =
        partial(sub_pointwise(cauchy_coefficient(re_seq(a), re_seq(b), n), cauchy_coefficient(im_seq(a), im_seq(b), n)), n.suc)
    partial_sub_pointwise(cauchy_coefficient(re_seq(a), re_seq(b), n), cauchy_coefficient(im_seq(a), im_seq(b), n), n.suc)
    partial(sub_pointwise(cauchy_coefficient(re_seq(a), re_seq(b), n), cauchy_coefficient(im_seq(a), im_seq(b), n)), n.suc) =
        partial(cauchy_coefficient(re_seq(a), re_seq(b), n), n.suc) - partial(cauchy_coefficient(im_seq(a), im_seq(b), n), n.suc)
    partial(re_seq(complex_cauchy_coefficient(a, b, n)), n.suc) =
        partial(cauchy_coefficient(re_seq(a), re_seq(b), n), n.suc) - partial(cauchy_coefficient(im_seq(a), im_seq(b), n), n.suc)
    cauchy_product(re_seq(a), re_seq(b), n) = partial(cauchy_coefficient(re_seq(a), re_seq(b), n), n.suc)
    cauchy_product(im_seq(a), im_seq(b), n) = partial(cauchy_coefficient(im_seq(a), im_seq(b), n), n.suc)
    re_seq(complex_cauchy_seq(a, b), n) =
        cauchy_product(re_seq(a), re_seq(b), n) - cauchy_product(im_seq(a), im_seq(b), n)
}

/// The imaginary part of the complex Cauchy product at index n is the sum of
/// the real Cauchy products of the mixed real and imaginary parts.
theorem complex_cauchy_product_im(a: Nat -> Complex, b: Nat -> Complex, n: Nat) {
    im_seq(complex_cauchy_seq(a, b), n) =
        cauchy_product(re_seq(a), im_seq(b), n) + cauchy_product(im_seq(a), re_seq(b), n)
} by {
    im_seq(complex_cauchy_seq(a, b), n) = complex_cauchy_seq(a, b, n).im
    complex_cauchy_seq(a, b, n) = complex_cauchy_product(a, b, n)
    complex_cauchy_seq(a, b, n).im = complex_cauchy_product(a, b, n).im
    complex_cauchy_product(a, b, n) = partial(complex_cauchy_coefficient(a, b, n), n.suc)
    complex_cauchy_product(a, b, n).im = partial(complex_cauchy_coefficient(a, b, n), n.suc).im
    partial_im_seq(complex_cauchy_coefficient(a, b, n), n.suc)
    partial(complex_cauchy_coefficient(a, b, n), n.suc).im =
        partial(im_seq(complex_cauchy_coefficient(a, b, n)), n.suc)
    forall(k: Nat) {
        if k < n.suc {
            complex_cauchy_coefficient_im(a, b, n, k)
            im_seq(complex_cauchy_coefficient(a, b, n), k) =
                cauchy_coefficient(re_seq(a), im_seq(b), n)(k) + cauchy_coefficient(im_seq(a), re_seq(b), n)(k)
            add_fn(cauchy_coefficient(re_seq(a), im_seq(b), n), cauchy_coefficient(im_seq(a), re_seq(b), n), k) =
                cauchy_coefficient(re_seq(a), im_seq(b), n)(k) + cauchy_coefficient(im_seq(a), re_seq(b), n)(k)
            im_seq(complex_cauchy_coefficient(a, b, n), k) =
                add_fn(cauchy_coefficient(re_seq(a), im_seq(b), n), cauchy_coefficient(im_seq(a), re_seq(b), n), k)
        }
    }
    partial_pointwise_eq(im_seq(complex_cauchy_coefficient(a, b, n)),
        add_fn(cauchy_coefficient(re_seq(a), im_seq(b), n), cauchy_coefficient(im_seq(a), re_seq(b), n)), n.suc)
    partial(im_seq(complex_cauchy_coefficient(a, b, n)), n.suc) =
        partial(add_fn(cauchy_coefficient(re_seq(a), im_seq(b), n), cauchy_coefficient(im_seq(a), re_seq(b), n)), n.suc)
    partial_add(cauchy_coefficient(re_seq(a), im_seq(b), n), cauchy_coefficient(im_seq(a), re_seq(b), n), n.suc)
    partial(add_fn(cauchy_coefficient(re_seq(a), im_seq(b), n), cauchy_coefficient(im_seq(a), re_seq(b), n)), n.suc) =
        partial(cauchy_coefficient(re_seq(a), im_seq(b), n), n.suc) + partial(cauchy_coefficient(im_seq(a), re_seq(b), n), n.suc)
    partial(im_seq(complex_cauchy_coefficient(a, b, n)), n.suc) =
        partial(cauchy_coefficient(re_seq(a), im_seq(b), n), n.suc) + partial(cauchy_coefficient(im_seq(a), re_seq(b), n), n.suc)
    cauchy_product(re_seq(a), im_seq(b), n) = partial(cauchy_coefficient(re_seq(a), im_seq(b), n), n.suc)
    cauchy_product(im_seq(a), re_seq(b), n) = partial(cauchy_coefficient(im_seq(a), re_seq(b), n), n.suc)
    im_seq(complex_cauchy_seq(a, b), n) =
        cauchy_product(re_seq(a), im_seq(b), n) + cauchy_product(im_seq(a), re_seq(b), n)
}

/// The real part of the partial sums of the complex Cauchy product is the
/// difference of the partial sums of the real Cauchy products.
theorem complex_cauchy_partial_re(a: Nat -> Complex, b: Nat -> Complex, n: Nat) {
    re_seq(partial(complex_cauchy_seq(a, b)), n) =
        partial(cauchy_seq(re_seq(a), re_seq(b)), n) - partial(cauchy_seq(im_seq(a), im_seq(b)), n)
} by {
    partial_re_seq(complex_cauchy_seq(a, b), n)
    re_seq(partial(complex_cauchy_seq(a, b)), n) = partial(re_seq(complex_cauchy_seq(a, b)), n)
    forall(i: Nat) {
        complex_cauchy_product_re(a, b, i)
        re_seq(complex_cauchy_seq(a, b), i) =
            cauchy_product(re_seq(a), re_seq(b), i) - cauchy_product(im_seq(a), im_seq(b), i)
        cauchy_seq(re_seq(a), re_seq(b), i) = cauchy_product(re_seq(a), re_seq(b), i)
        cauchy_seq(im_seq(a), im_seq(b), i) = cauchy_product(im_seq(a), im_seq(b), i)
        re_seq(complex_cauchy_seq(a, b), i) = cauchy_seq(re_seq(a), re_seq(b), i) - cauchy_seq(im_seq(a), im_seq(b), i)
        sub_pointwise(cauchy_seq(re_seq(a), re_seq(b)), cauchy_seq(im_seq(a), im_seq(b)), i) =
            cauchy_seq(re_seq(a), re_seq(b), i) - cauchy_seq(im_seq(a), im_seq(b), i)
        re_seq(complex_cauchy_seq(a, b), i) =
            sub_pointwise(cauchy_seq(re_seq(a), re_seq(b)), cauchy_seq(im_seq(a), im_seq(b)), i)
    }
    re_seq(complex_cauchy_seq(a, b)) =
        sub_pointwise(cauchy_seq(re_seq(a), re_seq(b)), cauchy_seq(im_seq(a), im_seq(b)))
    partial(re_seq(complex_cauchy_seq(a, b)), n) =
        partial(sub_pointwise(cauchy_seq(re_seq(a), re_seq(b)), cauchy_seq(im_seq(a), im_seq(b))), n)
    partial_sub_pointwise(cauchy_seq(re_seq(a), re_seq(b)), cauchy_seq(im_seq(a), im_seq(b)), n)
    partial(sub_pointwise(cauchy_seq(re_seq(a), re_seq(b)), cauchy_seq(im_seq(a), im_seq(b))), n) =
        partial(cauchy_seq(re_seq(a), re_seq(b)), n) - partial(cauchy_seq(im_seq(a), im_seq(b)), n)
    re_seq(partial(complex_cauchy_seq(a, b)), n) =
        partial(cauchy_seq(re_seq(a), re_seq(b)), n) - partial(cauchy_seq(im_seq(a), im_seq(b)), n)
}

/// The imaginary part of the partial sums of the complex Cauchy product is the
/// sum of the partial sums of the mixed real Cauchy products.
theorem complex_cauchy_partial_im(a: Nat -> Complex, b: Nat -> Complex, n: Nat) {
    im_seq(partial(complex_cauchy_seq(a, b)), n) =
        partial(cauchy_seq(re_seq(a), im_seq(b)), n) + partial(cauchy_seq(im_seq(a), re_seq(b)), n)
} by {
    partial_im_seq(complex_cauchy_seq(a, b), n)
    im_seq(partial(complex_cauchy_seq(a, b)), n) = partial(im_seq(complex_cauchy_seq(a, b)), n)
    forall(i: Nat) {
        complex_cauchy_product_im(a, b, i)
        im_seq(complex_cauchy_seq(a, b), i) =
            cauchy_product(re_seq(a), im_seq(b), i) + cauchy_product(im_seq(a), re_seq(b), i)
        cauchy_seq(re_seq(a), im_seq(b), i) = cauchy_product(re_seq(a), im_seq(b), i)
        cauchy_seq(im_seq(a), re_seq(b), i) = cauchy_product(im_seq(a), re_seq(b), i)
        im_seq(complex_cauchy_seq(a, b), i) = cauchy_seq(re_seq(a), im_seq(b), i) + cauchy_seq(im_seq(a), re_seq(b), i)
        add_fn(cauchy_seq(re_seq(a), im_seq(b)), cauchy_seq(im_seq(a), re_seq(b)), i) =
            cauchy_seq(re_seq(a), im_seq(b), i) + cauchy_seq(im_seq(a), re_seq(b), i)
        im_seq(complex_cauchy_seq(a, b), i) =
            add_fn(cauchy_seq(re_seq(a), im_seq(b)), cauchy_seq(im_seq(a), re_seq(b)), i)
    }
    im_seq(complex_cauchy_seq(a, b)) =
        add_fn(cauchy_seq(re_seq(a), im_seq(b)), cauchy_seq(im_seq(a), re_seq(b)))
    partial(im_seq(complex_cauchy_seq(a, b)), n) =
        partial(add_fn(cauchy_seq(re_seq(a), im_seq(b)), cauchy_seq(im_seq(a), re_seq(b))), n)
    partial_add(cauchy_seq(re_seq(a), im_seq(b)), cauchy_seq(im_seq(a), re_seq(b)), n)
    partial(add_fn(cauchy_seq(re_seq(a), im_seq(b)), cauchy_seq(im_seq(a), re_seq(b))), n) =
        partial(cauchy_seq(re_seq(a), im_seq(b)), n) + partial(cauchy_seq(im_seq(a), re_seq(b)), n)
    im_seq(partial(complex_cauchy_seq(a, b)), n) =
        partial(cauchy_seq(re_seq(a), im_seq(b)), n) + partial(cauchy_seq(im_seq(a), re_seq(b)), n)
}

// ---------------------------------------------------------------------------
// The binomial coefficient identity for the complex exponential series.
// (z, k).exp * (w, n - k).exp = C(n,k) * z^k * w^(n-k) / n!, so the Cauchy
// product of the two complex exponential series equals the exponential series
// of the sum termwise.
// ---------------------------------------------------------------------------

/// The embedding of a natural number as a complex number equals the embedding
/// of the real embedding of the natural number.
theorem complex_from_real_from_nat(m: Nat) {
    Complex.from_real(from_nat[Real](m)) = from_nat[Complex](m)
} by {
    define p(k: Nat) -> Bool {
        Complex.from_real(from_nat[Real](k)) = from_nat[Complex](k)
    }
    from_nat[Real](Nat.0) = Real.0
    from_nat[Complex](Nat.0) = Complex.0
    Complex.from_real(Real.0) = Complex.0
    Complex.from_real(from_nat[Real](Nat.0)) = from_nat[Complex](Nat.0)
    p(Nat.0)
    forall(k: Nat) {
        if p(k) {
            Complex.from_real(from_nat[Real](k)) = from_nat[Complex](k)
            from_nat[Real](k.suc) = from_nat[Real](k) + Real.1
            Complex.from_real(from_nat[Real](k.suc)) = Complex.from_real(from_nat[Real](k) + Real.1)
            real_add_lifts(from_nat[Real](k), Real.1)
            Complex.from_real(from_nat[Real](k) + Real.1) = Complex.from_real(from_nat[Real](k)) + Complex.from_real(Real.1)
            from_real_one
            Complex.from_real(Real.1) = Complex.1
            Complex.from_real(from_nat[Real](k.suc)) = Complex.from_real(from_nat[Real](k)) + Complex.1
            from_nat[Complex](k.suc) = from_nat[Complex](k) + Complex.1
            Complex.from_real(from_nat[Real](k.suc)) = from_nat[Complex](k.suc)
            p(k.suc)
        }
    }
    p(Nat.0) and forall(k: Nat) { p(k) implies p(k.suc) }
    alt_induction(p)
    p(m)
}

/// The complex version of the binomial coefficient identity:
/// 1 / (k! * (n-k)!) = C(n,k) / n! as complex numbers.
theorem complex_choose_factorial_inverse(n: Nat, k: Nat) {
    k <= n implies
    Complex.1 / (Complex.from_real(Real.from_rat(Rat.from_nat(k.factorial))) *
        Complex.from_real(Real.from_rat(Rat.from_nat((n - k).factorial)))) =
        from_nat[Complex](n.binom(k)) / Complex.from_real(Real.from_rat(Rat.from_nat(n.factorial)))
} by {
    if k <= n {
        choose_factorial_inverse(n, k)
        Real.1 / (Real.from_rat(Rat.from_nat(k.factorial)) * Real.from_rat(Rat.from_nat((n - k).factorial))) =
            from_nat[Real](n.binom(k)) / Real.from_rat(Rat.from_nat(n.factorial))
        Complex.from_real(Real.1 / (Real.from_rat(Rat.from_nat(k.factorial)) * Real.from_rat(Rat.from_nat((n - k).factorial)))) =
            Complex.from_real(from_nat[Real](n.binom(k)) / Real.from_rat(Rat.from_nat(n.factorial)))
        div_from_real(Real.1, Real.from_rat(Rat.from_nat(k.factorial)) * Real.from_rat(Rat.from_nat((n - k).factorial)))
        Complex.from_real(Real.1 / (Real.from_rat(Rat.from_nat(k.factorial)) * Real.from_rat(Rat.from_nat((n - k).factorial)))) =
            Complex.from_real(Real.1) / Complex.from_real(Real.from_rat(Rat.from_nat(k.factorial)) * Real.from_rat(Rat.from_nat((n - k).factorial)))
        from_real_one
        Complex.from_real(Real.1) = Complex.1
        real_mul_lifts(Real.from_rat(Rat.from_nat(k.factorial)), Real.from_rat(Rat.from_nat((n - k).factorial)))
        Complex.from_real(Real.from_rat(Rat.from_nat(k.factorial)) * Real.from_rat(Rat.from_nat((n - k).factorial))) =
            Complex.from_real(Real.from_rat(Rat.from_nat(k.factorial))) * Complex.from_real(Real.from_rat(Rat.from_nat((n - k).factorial)))
        Complex.from_real(Real.1 / (Real.from_rat(Rat.from_nat(k.factorial)) * Real.from_rat(Rat.from_nat((n - k).factorial)))) =
            Complex.1 / (Complex.from_real(Real.from_rat(Rat.from_nat(k.factorial))) *
                Complex.from_real(Real.from_rat(Rat.from_nat((n - k).factorial))))
        div_from_real(from_nat[Real](n.binom(k)), Real.from_rat(Rat.from_nat(n.factorial)))
        Complex.from_real(from_nat[Real](n.binom(k)) / Real.from_rat(Rat.from_nat(n.factorial))) =
            Complex.from_real(from_nat[Real](n.binom(k))) / Complex.from_real(Real.from_rat(Rat.from_nat(n.factorial)))
        complex_from_real_from_nat(n.binom(k))
        Complex.from_real(from_nat[Real](n.binom(k))) = from_nat[Complex](n.binom(k))
        Complex.from_real(from_nat[Real](n.binom(k)) / Real.from_rat(Rat.from_nat(n.factorial))) =
            from_nat[Complex](n.binom(k)) / Complex.from_real(Real.from_rat(Rat.from_nat(n.factorial)))
        Complex.1 / (Complex.from_real(Real.from_rat(Rat.from_nat(k.factorial))) *
            Complex.from_real(Real.from_rat(Rat.from_nat((n - k).factorial)))) =
            from_nat[Complex](n.binom(k)) / Complex.from_real(Real.from_rat(Rat.from_nat(n.factorial)))
    }
}

/// The product of two products reassociates and commutes:
/// (a * b) * (c * d) = (a * c) * (b * d).
theorem complex_mul_four_rearrange(a: Complex, b: Complex, c: Complex, d: Complex) {
    (a * b) * (c * d) = (a * c) * (b * d)
} by {
    mul_assoc(a, b, c * d)
    (a * b) * (c * d) = a * (b * (c * d))
    mul_assoc(b, c, d)
    b * (c * d) = (b * c) * d
    a * (b * (c * d)) = a * ((b * c) * d)
    mul_assoc(a, b * c, d)
    a * ((b * c) * d) = (a * (b * c)) * d
    (a * b) * (c * d) = (a * (b * c)) * d
    mul_comm(b, c)
    b * c = c * b
    a * (b * c) = a * (c * b)
    mul_assoc(a, c, b)
    a * (c * b) = (a * c) * b
    (a * (b * c)) * d = ((a * c) * b) * d
    mul_assoc(a * c, b, d)
    ((a * c) * b) * d = (a * c) * (b * d)
    (a * b) * (c * d) = (a * c) * (b * d)
}

/// Dividing one by a complex number is its inverse.
theorem complex_one_div(c: Complex) {
    Complex.1 / c = c.inverse
} by {
    Complex.1 / c = Complex.1 * c.inverse
    mul_one_left(c.inverse)
    Complex.1 * c.inverse = c.inverse
    Complex.1 / c = c.inverse
}

/// The product of two complex exponential terms is a combined fraction.
theorem complex_exp_term_product_combined(z: Complex, w: Complex, n: Nat, k: Nat) {
    k <= n implies
    complex_exp_term(z, k) * complex_exp_term(w, n - k) =
        (z.pow(k) * w.pow(n - k)) /
        (Complex.from_real(Real.from_rat(Rat.from_nat(k.factorial))) *
            Complex.from_real(Real.from_rat(Rat.from_nat((n - k).factorial))))
} by {
    if k <= n {
        complex_exp_term(z, k) = z.pow(k) / Complex.from_real(Real.from_rat(Rat.from_nat(k.factorial)))
        complex_exp_term(w, n - k) = w.pow(n - k) / Complex.from_real(Real.from_rat(Rat.from_nat((n - k).factorial)))
        complex_exp_term(z, k) * complex_exp_term(w, n - k) =
            (z.pow(k) / Complex.from_real(Real.from_rat(Rat.from_nat(k.factorial)))) *
            (w.pow(n - k) / Complex.from_real(Real.from_rat(Rat.from_nat((n - k).factorial))))
        (z.pow(k) / Complex.from_real(Real.from_rat(Rat.from_nat(k.factorial)))) *
            (w.pow(n - k) / Complex.from_real(Real.from_rat(Rat.from_nat((n - k).factorial)))) =
            (z.pow(k) * Complex.from_real(Real.from_rat(Rat.from_nat(k.factorial))).inverse) *
            (w.pow(n - k) * Complex.from_real(Real.from_rat(Rat.from_nat((n - k).factorial))).inverse)
        (z.pow(k) * Complex.from_real(Real.from_rat(Rat.from_nat(k.factorial))).inverse) *
            (w.pow(n - k) * Complex.from_real(Real.from_rat(Rat.from_nat((n - k).factorial))).inverse) =
            (z.pow(k) * w.pow(n - k)) *
            (Complex.from_real(Real.from_rat(Rat.from_nat(k.factorial))).inverse *
                Complex.from_real(Real.from_rat(Rat.from_nat((n - k).factorial))).inverse)
        complex_mul_four_rearrange(z.pow(k), Complex.from_real(Real.from_rat(Rat.from_nat(k.factorial))).inverse,
            w.pow(n - k), Complex.from_real(Real.from_rat(Rat.from_nat((n - k).factorial))).inverse)
        (z.pow(k) * Complex.from_real(Real.from_rat(Rat.from_nat(k.factorial))).inverse) *
            (w.pow(n - k) * Complex.from_real(Real.from_rat(Rat.from_nat((n - k).factorial))).inverse) =
            (z.pow(k) * w.pow(n - k)) *
            (Complex.from_real(Real.from_rat(Rat.from_nat(k.factorial))).inverse *
                Complex.from_real(Real.from_rat(Rat.from_nat((n - k).factorial))).inverse)
        inverse_dist[Complex](Complex.from_real(Real.from_rat(Rat.from_nat(k.factorial))),
            Complex.from_real(Real.from_rat(Rat.from_nat((n - k).factorial))))
        (Complex.from_real(Real.from_rat(Rat.from_nat(k.factorial))) *
            Complex.from_real(Real.from_rat(Rat.from_nat((n - k).factorial)))).inverse =
            Complex.from_real(Real.from_rat(Rat.from_nat(k.factorial))).inverse *
                Complex.from_real(Real.from_rat(Rat.from_nat((n - k).factorial))).inverse
        (z.pow(k) * w.pow(n - k)) *
            (Complex.from_real(Real.from_rat(Rat.from_nat(k.factorial))).inverse *
                Complex.from_real(Real.from_rat(Rat.from_nat((n - k).factorial))).inverse) =
            (z.pow(k) * w.pow(n - k)) *
            (Complex.from_real(Real.from_rat(Rat.from_nat(k.factorial))) *
                Complex.from_real(Real.from_rat(Rat.from_nat((n - k).factorial)))).inverse
        (z.pow(k) * w.pow(n - k)) *
            (Complex.from_real(Real.from_rat(Rat.from_nat(k.factorial))) *
                Complex.from_real(Real.from_rat(Rat.from_nat((n - k).factorial)))).inverse =
            (z.pow(k) * w.pow(n - k)) /
            (Complex.from_real(Real.from_rat(Rat.from_nat(k.factorial))) *
                Complex.from_real(Real.from_rat(Rat.from_nat((n - k).factorial))))
        complex_exp_term(z, k) * complex_exp_term(w, n - k) =
            (z.pow(k) * w.pow(n - k)) /
            (Complex.from_real(Real.from_rat(Rat.from_nat(k.factorial))) *
                Complex.from_real(Real.from_rat(Rat.from_nat((n - k).factorial))))
    }
}

/// The binomial fraction transform for complex numbers: the combined fraction
/// with denominator k! * (n-k)! becomes the binomial coefficient over n!.
theorem complex_binomial_fraction_transform(z: Complex, w: Complex, n: Nat, k: Nat) {
    k <= n implies
    (z.pow(k) * w.pow(n - k)) /
        (Complex.from_real(Real.from_rat(Rat.from_nat(k.factorial))) *
            Complex.from_real(Real.from_rat(Rat.from_nat((n - k).factorial)))) =
        (from_nat[Complex](n.binom(k)) * z.pow(k) * w.pow(n - k)) /
            Complex.from_real(Real.from_rat(Rat.from_nat(n.factorial)))
} by {
    if k <= n {
        complex_choose_factorial_inverse(n, k)
        Complex.1 / (Complex.from_real(Real.from_rat(Rat.from_nat(k.factorial))) *
            Complex.from_real(Real.from_rat(Rat.from_nat((n - k).factorial)))) =
            from_nat[Complex](n.binom(k)) / Complex.from_real(Real.from_rat(Rat.from_nat(n.factorial)))
        (z.pow(k) * w.pow(n - k)) *
            (Complex.1 / (Complex.from_real(Real.from_rat(Rat.from_nat(k.factorial))) *
                Complex.from_real(Real.from_rat(Rat.from_nat((n - k).factorial))))) =
            (z.pow(k) * w.pow(n - k)) *
            (from_nat[Complex](n.binom(k)) / Complex.from_real(Real.from_rat(Rat.from_nat(n.factorial))))
        complex_one_div(Complex.from_real(Real.from_rat(Rat.from_nat(k.factorial))) *
            Complex.from_real(Real.from_rat(Rat.from_nat((n - k).factorial))))
        Complex.1 / (Complex.from_real(Real.from_rat(Rat.from_nat(k.factorial))) *
            Complex.from_real(Real.from_rat(Rat.from_nat((n - k).factorial)))) =
            (Complex.from_real(Real.from_rat(Rat.from_nat(k.factorial))) *
                Complex.from_real(Real.from_rat(Rat.from_nat((n - k).factorial)))).inverse
        (z.pow(k) * w.pow(n - k)) *
            (Complex.1 / (Complex.from_real(Real.from_rat(Rat.from_nat(k.factorial))) *
                Complex.from_real(Real.from_rat(Rat.from_nat((n - k).factorial))))) =
            (z.pow(k) * w.pow(n - k)) *
            (Complex.from_real(Real.from_rat(Rat.from_nat(k.factorial))) *
                Complex.from_real(Real.from_rat(Rat.from_nat((n - k).factorial)))).inverse
        (z.pow(k) * w.pow(n - k)) *
            (Complex.from_real(Real.from_rat(Rat.from_nat(k.factorial))) *
                Complex.from_real(Real.from_rat(Rat.from_nat((n - k).factorial)))).inverse =
            (z.pow(k) * w.pow(n - k)) /
            (Complex.from_real(Real.from_rat(Rat.from_nat(k.factorial))) *
                Complex.from_real(Real.from_rat(Rat.from_nat((n - k).factorial))))
        (z.pow(k) * w.pow(n - k)) *
            (from_nat[Complex](n.binom(k)) / Complex.from_real(Real.from_rat(Rat.from_nat(n.factorial)))) =
            (z.pow(k) * w.pow(n - k)) *
            (from_nat[Complex](n.binom(k)) * Complex.from_real(Real.from_rat(Rat.from_nat(n.factorial))).inverse)
        mul_assoc(z.pow(k) * w.pow(n - k), from_nat[Complex](n.binom(k)),
            Complex.from_real(Real.from_rat(Rat.from_nat(n.factorial))).inverse)
        (z.pow(k) * w.pow(n - k)) *
            (from_nat[Complex](n.binom(k)) * Complex.from_real(Real.from_rat(Rat.from_nat(n.factorial))).inverse) =
            ((z.pow(k) * w.pow(n - k)) * from_nat[Complex](n.binom(k))) *
                Complex.from_real(Real.from_rat(Rat.from_nat(n.factorial))).inverse
        mul_comm(z.pow(k) * w.pow(n - k), from_nat[Complex](n.binom(k)))
        (z.pow(k) * w.pow(n - k)) * from_nat[Complex](n.binom(k)) =
            from_nat[Complex](n.binom(k)) * (z.pow(k) * w.pow(n - k))
        ((z.pow(k) * w.pow(n - k)) * from_nat[Complex](n.binom(k))) *
            Complex.from_real(Real.from_rat(Rat.from_nat(n.factorial))).inverse =
            (from_nat[Complex](n.binom(k)) * (z.pow(k) * w.pow(n - k))) *
                Complex.from_real(Real.from_rat(Rat.from_nat(n.factorial))).inverse
        mul_assoc(from_nat[Complex](n.binom(k)), z.pow(k), w.pow(n - k))
        from_nat[Complex](n.binom(k)) * (z.pow(k) * w.pow(n - k)) =
            (from_nat[Complex](n.binom(k)) * z.pow(k)) * w.pow(n - k)
        (from_nat[Complex](n.binom(k)) * (z.pow(k) * w.pow(n - k))) *
            Complex.from_real(Real.from_rat(Rat.from_nat(n.factorial))).inverse =
            (from_nat[Complex](n.binom(k)) * z.pow(k) * w.pow(n - k)) *
                Complex.from_real(Real.from_rat(Rat.from_nat(n.factorial))).inverse
        (from_nat[Complex](n.binom(k)) * z.pow(k) * w.pow(n - k)) *
            Complex.from_real(Real.from_rat(Rat.from_nat(n.factorial))).inverse =
            (from_nat[Complex](n.binom(k)) * z.pow(k) * w.pow(n - k)) /
                Complex.from_real(Real.from_rat(Rat.from_nat(n.factorial)))
        (z.pow(k) * w.pow(n - k)) /
            (Complex.from_real(Real.from_rat(Rat.from_nat(k.factorial))) *
                Complex.from_real(Real.from_rat(Rat.from_nat((n - k).factorial)))) =
            (from_nat[Complex](n.binom(k)) * z.pow(k) * w.pow(n - k)) /
                Complex.from_real(Real.from_rat(Rat.from_nat(n.factorial)))
    }
}

/// The product of two complex exponential terms equals the binomial term over n!.
theorem complex_exp_term_product_binomial(z: Complex, w: Complex, n: Nat, k: Nat) {
    k <= n implies
    complex_exp_term(z, k) * complex_exp_term(w, n - k) =
        binomial_term[Complex](z, w, n, k) / Complex.from_real(Real.from_rat(Rat.from_nat(n.factorial)))
} by {
    if k <= n {
        complex_exp_term_product_combined(z, w, n, k)
        complex_exp_term(z, k) * complex_exp_term(w, n - k) =
            (z.pow(k) * w.pow(n - k)) /
            (Complex.from_real(Real.from_rat(Rat.from_nat(k.factorial))) *
                Complex.from_real(Real.from_rat(Rat.from_nat((n - k).factorial))))
        complex_binomial_fraction_transform(z, w, n, k)
        (z.pow(k) * w.pow(n - k)) /
            (Complex.from_real(Real.from_rat(Rat.from_nat(k.factorial))) *
                Complex.from_real(Real.from_rat(Rat.from_nat((n - k).factorial)))) =
            (from_nat[Complex](n.binom(k)) * z.pow(k) * w.pow(n - k)) /
                Complex.from_real(Real.from_rat(Rat.from_nat(n.factorial)))
        binomial_term[Complex](z, w, n, k) = from_nat[Complex](n.binom(k)) * z.pow(k) * w.pow(n - k)
        (from_nat[Complex](n.binom(k)) * z.pow(k) * w.pow(n - k)) /
            Complex.from_real(Real.from_rat(Rat.from_nat(n.factorial))) =
            binomial_term[Complex](z, w, n, k) / Complex.from_real(Real.from_rat(Rat.from_nat(n.factorial)))
        complex_exp_term(z, k) * complex_exp_term(w, n - k) =
            binomial_term[Complex](z, w, n, k) / Complex.from_real(Real.from_rat(Rat.from_nat(n.factorial)))
    }
}

/// Dividing a sequence by a constant pointwise.
define complex_div_fn(f: Nat -> Complex, c: Complex, n: Nat) -> Complex {
    f(n) / c
}

/// The partial sum of a sequence divided by a constant is the partial sum
/// divided by the constant.
theorem complex_partial_div_fn(f: Nat -> Complex, c: Complex, n: Nat) {
    partial(complex_div_fn(f, c), n) = partial(f, n) / c
} by {
    forall(k: Nat) {
        complex_div_fn(f, c, k) = f(k) / c
        f(k) / c = f(k) * c.inverse
        mul_fn(c.inverse, f, k) = c.inverse * f(k)
        c.inverse * f(k) = f(k) * c.inverse
        complex_div_fn(f, c, k) = mul_fn(c.inverse, f, k)
    }
    complex_div_fn(f, c) = mul_fn(c.inverse, f)
    partial_scalar_mul(c.inverse, f, n)
    c.inverse * partial(f, n) = partial(mul_fn(c.inverse, f), n)
    partial(complex_div_fn(f, c), n) = c.inverse * partial(f, n)
    mul_comm(c.inverse, partial(f, n))
    c.inverse * partial(f, n) = partial(f, n) * c.inverse
    partial(f, n) * c.inverse = partial(f, n) / c
    partial(complex_div_fn(f, c), n) = partial(f, n) / c
}

/// The Cauchy product of the complex exponential series at z and w equals the
/// complex exponential series at z + w termwise.
theorem complex_exp_cauchy_product_eq(z: Complex, w: Complex, n: Nat) {
    complex_cauchy_product(complex_exp_term(z), complex_exp_term(w), n) = complex_exp_term(z + w, n)
} by {
    forall(k: Nat) {
        if k < n.suc {
            k <= n
            complex_exp_term_product_binomial(z, w, n, k)
            complex_exp_term(z, k) * complex_exp_term(w, n - k) =
                binomial_term[Complex](z, w, n, k) / Complex.from_real(Real.from_rat(Rat.from_nat(n.factorial)))
            complex_div_fn(binomial_term[Complex](z, w, n), Complex.from_real(Real.from_rat(Rat.from_nat(n.factorial))), k) =
                binomial_term[Complex](z, w, n, k) / Complex.from_real(Real.from_rat(Rat.from_nat(n.factorial)))
            complex_exp_term(z, k) * complex_exp_term(w, n - k) =
                complex_div_fn(binomial_term[Complex](z, w, n), Complex.from_real(Real.from_rat(Rat.from_nat(n.factorial))), k)
        }
    }
    define f(k: Nat) -> Complex {
        complex_exp_term(z, k) * complex_exp_term(w, n - k)
    }
    define g(k: Nat) -> Complex {
        complex_div_fn(binomial_term[Complex](z, w, n), Complex.from_real(Real.from_rat(Rat.from_nat(n.factorial))), k)
    }
    forall(k: Nat) {
        if k < n.suc {
            complex_exp_term(z, k) * complex_exp_term(w, n - k) =
                complex_div_fn(binomial_term[Complex](z, w, n), Complex.from_real(Real.from_rat(Rat.from_nat(n.factorial))), k)
            f(k) = g(k)
        }
    }
    partial_pointwise_eq(f, g, n.suc)
    partial(f, n.suc) = partial(g, n.suc)
    partial(g, n.suc) =
        partial(complex_div_fn(binomial_term[Complex](z, w, n), Complex.from_real(Real.from_rat(Rat.from_nat(n.factorial)))), n.suc)
    complex_partial_div_fn(binomial_term[Complex](z, w, n), Complex.from_real(Real.from_rat(Rat.from_nat(n.factorial))), n.suc)
    partial(complex_div_fn(binomial_term[Complex](z, w, n), Complex.from_real(Real.from_rat(Rat.from_nat(n.factorial)))), n.suc) =
        partial(binomial_term[Complex](z, w, n), n.suc) / Complex.from_real(Real.from_rat(Rat.from_nat(n.factorial)))
    binomial[Complex](z, w, n)
    (z + w).pow(n) = partial(binomial_term[Complex](z, w, n), n.suc)
    partial(binomial_term[Complex](z, w, n), n.suc) / Complex.from_real(Real.from_rat(Rat.from_nat(n.factorial))) =
        (z + w).pow(n) / Complex.from_real(Real.from_rat(Rat.from_nat(n.factorial)))
    partial(f, n.suc) = (z + w).pow(n) / Complex.from_real(Real.from_rat(Rat.from_nat(n.factorial)))
    define cc(k: Nat) -> Complex {
        complex_cauchy_coefficient(complex_exp_term(z), complex_exp_term(w), n)(k)
    }
    forall(k: Nat) {
        if k < n.suc {
            complex_cauchy_coefficient(complex_exp_term(z), complex_exp_term(w), n, k) =
                complex_exp_term(z, k) * complex_exp_term(w, n - k)
            cc(k) = complex_cauchy_coefficient(complex_exp_term(z), complex_exp_term(w), n, k)
            cc(k) = complex_exp_term(z, k) * complex_exp_term(w, n - k)
            cc(k) = f(k)
        }
    }
    partial_pointwise_eq(cc, f, n.suc)
    partial(cc, n.suc) = partial(f, n.suc)
    complex_cauchy_product(complex_exp_term(z), complex_exp_term(w), n) =
        partial(complex_cauchy_coefficient(complex_exp_term(z), complex_exp_term(w), n), n.suc)
    partial(complex_cauchy_coefficient(complex_exp_term(z), complex_exp_term(w), n), n.suc) = partial(cc, n.suc)
    complex_cauchy_product(complex_exp_term(z), complex_exp_term(w), n) = partial(f, n.suc)
    complex_cauchy_product(complex_exp_term(z), complex_exp_term(w), n) =
        (z + w).pow(n) / Complex.from_real(Real.from_rat(Rat.from_nat(n.factorial)))
    complex_exp_term(z + w, n) = (z + w).pow(n) / Complex.from_real(Real.from_rat(Rat.from_nat(n.factorial)))
    complex_cauchy_product(complex_exp_term(z), complex_exp_term(w), n) = complex_exp_term(z + w, n)
}

/// The complex Cauchy product sequence of the two complex exponential series is
/// the complex exponential series of the sum.
theorem complex_exp_cauchy_seq_eq(z: Complex, w: Complex) {
    complex_cauchy_seq(complex_exp_term(z), complex_exp_term(w)) = complex_exp_term(z + w)
} by {
    forall(n: Nat) {
        complex_cauchy_seq(complex_exp_term(z), complex_exp_term(w), n) =
            complex_cauchy_product(complex_exp_term(z), complex_exp_term(w), n)
        complex_exp_cauchy_product_eq(z, w, n)
        complex_cauchy_product(complex_exp_term(z), complex_exp_term(w), n) = complex_exp_term(z + w, n)
        complex_cauchy_seq(complex_exp_term(z), complex_exp_term(w), n) = complex_exp_term(z + w, n)
    }
}

// ---------------------------------------------------------------------------
// The functional equation (z + w).exp = z.exp * w.exp.  The complex Cauchy
// product of the two exponential series converges to the product of the two
// exponentials, and the Cauchy product equals the exponential series of the
// sum termwise, so both limits agree.
// ---------------------------------------------------------------------------

/// The real-part sequence of the complex exponential terms.
define ce_re(z: Complex, n: Nat) -> Real {
    re_seq(complex_exp_term(z), n)
}

/// The imaginary-part sequence of the complex exponential terms.
define ce_im(z: Complex, n: Nat) -> Real {
    im_seq(complex_exp_term(z), n)
}

/// The absolute value of each element in a sequence equals the pointwise
/// absolute value sequence.
theorem seq_abs_eq_abs_fn(a: Nat -> Real) {
    seq_abs(a) = abs_fn(a)
} by {
    forall(n: Nat) {
        seq_abs(a, n) = a(n).abs
        abs_fn(a, n) = a(n).abs
        seq_abs(a, n) = abs_fn(a, n)
    }
}

/// The real-part sequence of the complex exponential series is absolutely convergent.
theorem complex_exp_series_re_absolutely_converges(z: Complex) {
    absolutely_converges(ce_re(z))
} by {
    seq_abs_eq_abs_fn(ce_re(z))
    seq_abs(ce_re(z)) = abs_fn(ce_re(z))
    complex_exp_series_re_abs_conv(z)
    converges(partial(seq_abs(re_seq(complex_exp_term(z)))))
    converges(partial(seq_abs(ce_re(z))))
    converges(partial(abs_fn(ce_re(z))))
    absolutely_converges(ce_re(z))
}

/// The imaginary-part sequence of the complex exponential series is absolutely convergent.
theorem complex_exp_series_im_absolutely_converges(z: Complex) {
    absolutely_converges(ce_im(z))
} by {
    seq_abs_eq_abs_fn(ce_im(z))
    seq_abs(ce_im(z)) = abs_fn(ce_im(z))
    complex_exp_series_im_abs_conv(z)
    converges(partial(seq_abs(im_seq(complex_exp_term(z)))))
    converges(partial(seq_abs(ce_im(z))))
    converges(partial(abs_fn(ce_im(z))))
    absolutely_converges(ce_im(z))
}

/// The limit of the partial sums of the real parts of the complex exponential
/// series is the real part of the complex exponential.
theorem complex_exp_re_partial_limit(z: Complex) {
    limit(partial(ce_re(z))) = complex_exp(z).re
} by {
    complex_exp_series_converges(z)
    complex_converges(partial(complex_exp_term(z)))
    complex_limit_re(partial(complex_exp_term(z)))
    limit(re_seq(partial(complex_exp_term(z)))) = complex_limit(partial(complex_exp_term(z))).re
    complex_exp(z) = complex_limit(partial(complex_exp_term(z)))
    complex_exp(z).re = complex_limit(partial(complex_exp_term(z))).re
    limit(re_seq(partial(complex_exp_term(z)))) = complex_exp(z).re
    forall(n: Nat) {
        partial_re_seq(complex_exp_term(z), n)
        re_seq(partial(complex_exp_term(z)), n) = partial(re_seq(complex_exp_term(z)), n)
        re_seq(partial(complex_exp_term(z)), n) = partial(ce_re(z), n)
    }
    re_seq(partial(complex_exp_term(z))) = partial(ce_re(z))
    limit(re_seq(partial(complex_exp_term(z)))) = limit(partial(ce_re(z)))
    limit(partial(ce_re(z))) = complex_exp(z).re
}

/// The limit of the partial sums of the imaginary parts of the complex
/// exponential series is the imaginary part of the complex exponential.
theorem complex_exp_im_partial_limit(z: Complex) {
    limit(partial(ce_im(z))) = complex_exp(z).im
} by {
    complex_exp_series_converges(z)
    complex_converges(partial(complex_exp_term(z)))
    complex_limit_im(partial(complex_exp_term(z)))
    limit(im_seq(partial(complex_exp_term(z)))) = complex_limit(partial(complex_exp_term(z))).im
    complex_exp(z) = complex_limit(partial(complex_exp_term(z)))
    complex_exp(z).im = complex_limit(partial(complex_exp_term(z))).im
    limit(im_seq(partial(complex_exp_term(z)))) = complex_exp(z).im
    forall(n: Nat) {
        partial_im_seq(complex_exp_term(z), n)
        im_seq(partial(complex_exp_term(z)), n) = partial(im_seq(complex_exp_term(z)), n)
        im_seq(partial(complex_exp_term(z)), n) = partial(ce_im(z), n)
    }
    im_seq(partial(complex_exp_term(z))) = partial(ce_im(z))
    limit(im_seq(partial(complex_exp_term(z)))) = limit(partial(ce_im(z)))
    limit(partial(ce_im(z))) = complex_exp(z).im
}

/// The partial sums of the complex Cauchy product of the two complex
/// exponential series converge to the product of the two complex exponentials.
theorem complex_exp_cauchy_seq_converges_to(z: Complex, w: Complex) {
    complex_converges_to(partial(complex_cauchy_seq(complex_exp_term(z), complex_exp_term(w))),
        complex_exp(z) * complex_exp(w))
} by {
    complex_exp_series_re_absolutely_converges(z)
    complex_exp_series_im_absolutely_converges(z)
    complex_exp_series_re_absolutely_converges(w)
    complex_exp_series_im_absolutely_converges(w)
    absolutely_converges(ce_re(z))
    absolutely_converges(ce_im(z))
    absolutely_converges(ce_re(w))
    absolutely_converges(ce_im(w))

    cauchy_product_converges(ce_re(z), ce_re(w))
    converges_to(partial(cauchy_seq(ce_re(z), ce_re(w))),
        limit(partial(ce_re(z))) * limit(partial(ce_re(w))))
    cauchy_product_converges(ce_im(z), ce_im(w))
    converges_to(partial(cauchy_seq(ce_im(z), ce_im(w))),
        limit(partial(ce_im(z))) * limit(partial(ce_im(w))))
    cauchy_product_converges(ce_re(z), ce_im(w))
    converges_to(partial(cauchy_seq(ce_re(z), ce_im(w))),
        limit(partial(ce_re(z))) * limit(partial(ce_im(w))))
    cauchy_product_converges(ce_im(z), ce_re(w))
    converges_to(partial(cauchy_seq(ce_im(z), ce_re(w))),
        limit(partial(ce_im(z))) * limit(partial(ce_re(w))))

    forall(n: Nat) {
        complex_cauchy_partial_re(complex_exp_term(z), complex_exp_term(w), n)
        re_seq(partial(complex_cauchy_seq(complex_exp_term(z), complex_exp_term(w))), n) =
            partial(cauchy_seq(re_seq(complex_exp_term(z)), re_seq(complex_exp_term(w))), n) -
            partial(cauchy_seq(im_seq(complex_exp_term(z)), im_seq(complex_exp_term(w))), n)
        re_seq(partial(complex_cauchy_seq(complex_exp_term(z), complex_exp_term(w))), n) =
            partial(cauchy_seq(ce_re(z), ce_re(w)), n) - partial(cauchy_seq(ce_im(z), ce_im(w)), n)
        sub_pointwise(partial(cauchy_seq(ce_re(z), ce_re(w))), partial(cauchy_seq(ce_im(z), ce_im(w))), n) =
            partial(cauchy_seq(ce_re(z), ce_re(w)), n) - partial(cauchy_seq(ce_im(z), ce_im(w)), n)
        re_seq(partial(complex_cauchy_seq(complex_exp_term(z), complex_exp_term(w))), n) =
            sub_pointwise(partial(cauchy_seq(ce_re(z), ce_re(w))), partial(cauchy_seq(ce_im(z), ce_im(w))), n)
    }
    sub_pointwise_converges_to(partial(cauchy_seq(ce_re(z), ce_re(w))),
        partial(cauchy_seq(ce_im(z), ce_im(w))),
        re_seq(partial(complex_cauchy_seq(complex_exp_term(z), complex_exp_term(w)))),
        limit(partial(ce_re(z))) * limit(partial(ce_re(w))),
        limit(partial(ce_im(z))) * limit(partial(ce_im(w))))
    converges_to(re_seq(partial(complex_cauchy_seq(complex_exp_term(z), complex_exp_term(w)))),
        limit(partial(ce_re(z))) * limit(partial(ce_re(w))) -
        limit(partial(ce_im(z))) * limit(partial(ce_im(w))))

    complex_exp_re_partial_limit(z)
    complex_exp_im_partial_limit(z)
    complex_exp_re_partial_limit(w)
    complex_exp_im_partial_limit(w)
    limit(partial(ce_re(z))) = complex_exp(z).re
    limit(partial(ce_im(z))) = complex_exp(z).im
    limit(partial(ce_re(w))) = complex_exp(w).re
    limit(partial(ce_im(w))) = complex_exp(w).im
    limit(partial(ce_re(z))) * limit(partial(ce_re(w))) -
        limit(partial(ce_im(z))) * limit(partial(ce_im(w))) =
        complex_exp(z).re * complex_exp(w).re - complex_exp(z).im * complex_exp(w).im
    re_mul(complex_exp(z), complex_exp(w))
    (complex_exp(z) * complex_exp(w)).re =
        complex_exp(z).re * complex_exp(w).re - complex_exp(z).im * complex_exp(w).im
    converges_to(re_seq(partial(complex_cauchy_seq(complex_exp_term(z), complex_exp_term(w)))),
        (complex_exp(z) * complex_exp(w)).re)

    forall(n: Nat) {
        complex_cauchy_partial_im(complex_exp_term(z), complex_exp_term(w), n)
        im_seq(partial(complex_cauchy_seq(complex_exp_term(z), complex_exp_term(w))), n) =
            partial(cauchy_seq(re_seq(complex_exp_term(z)), im_seq(complex_exp_term(w))), n) +
            partial(cauchy_seq(im_seq(complex_exp_term(z)), re_seq(complex_exp_term(w))), n)
        im_seq(partial(complex_cauchy_seq(complex_exp_term(z), complex_exp_term(w))), n) =
            partial(cauchy_seq(ce_re(z), ce_im(w)), n) + partial(cauchy_seq(ce_im(z), ce_re(w)), n)
        add_fn(partial(cauchy_seq(ce_re(z), ce_im(w))), partial(cauchy_seq(ce_im(z), ce_re(w))), n) =
            partial(cauchy_seq(ce_re(z), ce_im(w)), n) + partial(cauchy_seq(ce_im(z), ce_re(w)), n)
        im_seq(partial(complex_cauchy_seq(complex_exp_term(z), complex_exp_term(w))), n) =
            add_fn(partial(cauchy_seq(ce_re(z), ce_im(w))), partial(cauchy_seq(ce_im(z), ce_re(w))), n)
    }
    add_pointwise_converges_to(partial(cauchy_seq(ce_re(z), ce_im(w))),
        partial(cauchy_seq(ce_im(z), ce_re(w))),
        im_seq(partial(complex_cauchy_seq(complex_exp_term(z), complex_exp_term(w)))),
        limit(partial(ce_re(z))) * limit(partial(ce_im(w))),
        limit(partial(ce_im(z))) * limit(partial(ce_re(w))))
    converges_to(im_seq(partial(complex_cauchy_seq(complex_exp_term(z), complex_exp_term(w)))),
        limit(partial(ce_re(z))) * limit(partial(ce_im(w))) +
        limit(partial(ce_im(z))) * limit(partial(ce_re(w))))
    limit(partial(ce_re(z))) * limit(partial(ce_im(w))) +
        limit(partial(ce_im(z))) * limit(partial(ce_re(w))) =
        complex_exp(z).re * complex_exp(w).im + complex_exp(z).im * complex_exp(w).re
    im_mul(complex_exp(z), complex_exp(w))
    (complex_exp(z) * complex_exp(w)).im =
        complex_exp(z).re * complex_exp(w).im + complex_exp(z).im * complex_exp(w).re
    converges_to(im_seq(partial(complex_cauchy_seq(complex_exp_term(z), complex_exp_term(w)))),
        (complex_exp(z) * complex_exp(w)).im)

    componentwise_imp_complex_converges_to(partial(complex_cauchy_seq(complex_exp_term(z), complex_exp_term(w))),
        complex_exp(z) * complex_exp(w))
}

/// The complex exponential satisfies the functional equation:
/// (z + w).exp = z.exp * w.exp.
theorem complex_exp_add(z: Complex, w: Complex) {
    complex_exp(z + w) = complex_exp(z) * complex_exp(w)
} by {
    complex_exp_cauchy_seq_converges_to(z, w)
    complex_converges_to(partial(complex_cauchy_seq(complex_exp_term(z), complex_exp_term(w))),
        complex_exp(z) * complex_exp(w))
    complex_exp_cauchy_seq_eq(z, w)
    complex_cauchy_seq(complex_exp_term(z), complex_exp_term(w)) = complex_exp_term(z + w)
    partial(complex_cauchy_seq(complex_exp_term(z), complex_exp_term(w))) = partial(complex_exp_term(z + w))
    complex_converges_to(partial(complex_exp_term(z + w)), complex_exp(z) * complex_exp(w))
    complex_converges_to_imp_limit(partial(complex_exp_term(z + w)), complex_exp(z) * complex_exp(w))
    complex_limit(partial(complex_exp_term(z + w))) = complex_exp(z) * complex_exp(w)
    complex_exp(z + w) = complex_limit(partial(complex_exp_term(z + w)))
    complex_exp(z + w) = complex_exp(z) * complex_exp(w)
}

// ---------------------------------------------------------------------------
// Euler's identity: (i·x).exp = x.cos + i·x.sin, hence (i·pi).exp = -1.
// The complex exponential series at i·x splits into the real cosine series and
// the imaginary sine series, whose even-indexed partial sums are the partial
// sums of the real Real.cos and Real.sin series.
// ---------------------------------------------------------------------------

/// The imaginary unit times a real number.
define complex_i_mul_real(x: Real) -> Complex {
    Complex.i * Complex.from_real(x)
}

/// The complex number (-1, 0) is negative one.
theorem complex_neg_one_new {
    Complex.new(-Real.1, Real.0) = -Complex.1
} by {
    neg_one_lifts
    -Complex.1 = Complex.from_real(-Real.1)
    Complex.from_real(-Real.1) = Complex.new(-Real.1, Real.0)
    Complex.new(-Real.1, Real.0) = -Complex.1
}

/// The square of i times a real number: (i·a)·(i·x) = -(a·x).
theorem complex_i_mul_real_square(a: Real, x: Real) {
    (Complex.i * Complex.from_real(a)) * (Complex.i * Complex.from_real(x)) =
        -Complex.from_real(a * x)
} by {
    complex_mul_four_rearrange(Complex.i, Complex.from_real(a), Complex.i, Complex.from_real(x))
    (Complex.i * Complex.from_real(a)) * (Complex.i * Complex.from_real(x)) =
        (Complex.i * Complex.i) * (Complex.from_real(a) * Complex.from_real(x))
    i_squared
    Complex.i * Complex.i = Complex.new(-Real.1, Real.0)
    complex_neg_one_new
    Complex.new(-Real.1, Real.0) = -Complex.1
    (Complex.i * Complex.i) * (Complex.from_real(a) * Complex.from_real(x)) =
        (-Complex.1) * (Complex.from_real(a) * Complex.from_real(x))
    real_mul_lifts(a, x)
    Complex.from_real(a * x) = Complex.from_real(a) * Complex.from_real(x)
    (Complex.from_real(a) * Complex.from_real(x)) = Complex.from_real(a * x)
    (-Complex.1) * (Complex.from_real(a) * Complex.from_real(x)) = (-Complex.1) * Complex.from_real(a * x)
    neg_eq_mul_neg_one(Complex.from_real(a * x))
    -Complex.from_real(a * x) = (-Complex.1) * Complex.from_real(a * x)
    (-Complex.1) * Complex.from_real(a * x) = -Complex.from_real(a * x)
    (Complex.i * Complex.from_real(a)) * (Complex.i * Complex.from_real(x)) =
        -Complex.from_real(a * x)
}

/// A real number times i times a real number is i times their product.
theorem complex_from_real_mul_i(a: Real, x: Real) {
    Complex.from_real(a) * (Complex.i * Complex.from_real(x)) =
        Complex.i * Complex.from_real(a * x)
} by {
    mul_assoc(Complex.from_real(a), Complex.i, Complex.from_real(x))
    Complex.from_real(a) * (Complex.i * Complex.from_real(x)) =
        (Complex.from_real(a) * Complex.i) * Complex.from_real(x)
    mul_comm(Complex.from_real(a), Complex.i)
    Complex.from_real(a) * Complex.i = Complex.i * Complex.from_real(a)
    (Complex.from_real(a) * Complex.i) * Complex.from_real(x) =
        (Complex.i * Complex.from_real(a)) * Complex.from_real(x)
    mul_assoc(Complex.i, Complex.from_real(a), Complex.from_real(x))
    (Complex.i * Complex.from_real(a)) * Complex.from_real(x) =
        Complex.i * (Complex.from_real(a) * Complex.from_real(x))
    real_mul_lifts(a, x)
    Complex.from_real(a * x) = Complex.from_real(a) * Complex.from_real(x)
    Complex.i * (Complex.from_real(a) * Complex.from_real(x)) = Complex.i * Complex.from_real(a * x)
    Complex.from_real(a) * (Complex.i * Complex.from_real(x)) =
        Complex.i * Complex.from_real(a * x)
}

/// Real multiplication is associative.
theorem real_mul_assoc(a: Real, b: Real, c: Real) {
    (a * b) * c = a * (b * c)
} by {
}

/// One is a left identity for real multiplication.
theorem real_mul_one_left(a: Real) {
    Real.1 * a = a
} by {
}

/// One is a right identity for real multiplication.
theorem real_mul_one_right(a: Real) {
    a * Real.1 = a
} by {
}

/// The powers of i·x alternate between real and imaginary:
/// (i·x)^(2k) = (-1)^k x^(2k) and (i·x)^(2k+1) = i·(-1)^k x^(2k+1).
theorem complex_i_mul_real_pow_parity(x: Real, k: Nat) {
    complex_i_mul_real(x).pow(Nat.2 * k) = Complex.from_real(alternating_sign[Real](k) * x.pow(Nat.2 * k))
        and
    complex_i_mul_real(x).pow(Nat.2 * k + Nat.1) = Complex.i * Complex.from_real(alternating_sign[Real](k) * x.pow(Nat.2 * k + Nat.1))
} by {
    define p(m: Nat) -> Bool {
        complex_i_mul_real(x).pow(Nat.2 * m) = Complex.from_real(alternating_sign[Real](m) * x.pow(Nat.2 * m))
            and
        complex_i_mul_real(x).pow(Nat.2 * m + Nat.1) = Complex.i * Complex.from_real(alternating_sign[Real](m) * x.pow(Nat.2 * m + Nat.1))
    }
    pow_zero(complex_i_mul_real(x))
    complex_i_mul_real(x).pow(Nat.0) = Complex.1
    alternating_sign_zero[Real]
    alternating_sign[Real](Nat.0) = Real.1
    pow_zero(x)
    x.pow(Nat.0) = Real.1
    Real.1 * Real.1 = Real.1
    Complex.from_real(Real.1 * Real.1) = Complex.from_real(Real.1)
    from_real_one
    Complex.from_real(Real.1) = Complex.1
    Complex.from_real(Real.1 * Real.1) = Complex.1
    Complex.from_real(alternating_sign[Real](Nat.0) * x.pow(Nat.0)) = Complex.1
    Nat.2 * Nat.0 = Nat.0
    complex_i_mul_real(x).pow(Nat.2 * Nat.0) = Complex.from_real(alternating_sign[Real](Nat.0) * x.pow(Nat.2 * Nat.0))
    complex_pow_suc(complex_i_mul_real(x), Nat.0)
    complex_i_mul_real(x).pow(Nat.0.suc) = complex_i_mul_real(x) * complex_i_mul_real(x).pow(Nat.0)
    complex_i_mul_real(x).pow(Nat.0.suc) = complex_i_mul_real(x) * Complex.1
    mul_one_right(complex_i_mul_real(x))
    complex_i_mul_real(x) * Complex.1 = complex_i_mul_real(x)
    complex_i_mul_real(x).pow(Nat.0.suc) = complex_i_mul_real(x)
    complex_i_mul_real(x) = Complex.i * Complex.from_real(x)
    complex_i_mul_real(x).pow(Nat.0.suc) = Complex.i * Complex.from_real(x)
    pow_one(x)
    x.pow(Nat.1) = x
    Nat.2 * Nat.0 + Nat.1 = Nat.1
    Nat.0.suc = Nat.1
    alternating_sign[Real](Nat.0) * x.pow(Nat.2 * Nat.0 + Nat.1) = x
    Complex.i * Complex.from_real(alternating_sign[Real](Nat.0) * x.pow(Nat.2 * Nat.0 + Nat.1)) =
        Complex.i * Complex.from_real(x)
    complex_i_mul_real(x).pow(Nat.2 * Nat.0 + Nat.1) = Complex.i * Complex.from_real(alternating_sign[Real](Nat.0) * x.pow(Nat.2 * Nat.0 + Nat.1))
    p(Nat.0)
    forall(m: Nat) {
        if p(m) {
            complex_i_mul_real(x).pow(Nat.2 * m) = Complex.from_real(alternating_sign[Real](m) * x.pow(Nat.2 * m))
            complex_i_mul_real(x).pow(Nat.2 * m + Nat.1) = Complex.i * Complex.from_real(alternating_sign[Real](m) * x.pow(Nat.2 * m + Nat.1))

            // even part of p(m.suc):
            two_mul_suc(m)
            Nat.2 * m.suc = (Nat.2 * m).suc.suc
            complex_pow_suc(complex_i_mul_real(x), Nat.2 * m + Nat.1)
            complex_i_mul_real(x).pow((Nat.2 * m + Nat.1).suc) = complex_i_mul_real(x) * complex_i_mul_real(x).pow(Nat.2 * m + Nat.1)
            (Nat.2 * m + Nat.1).suc = Nat.2 * m + Nat.2
            Nat.2 * m + Nat.2 = (Nat.2 * m).suc.suc
            complex_i_mul_real(x).pow(Nat.2 * m.suc) = complex_i_mul_real(x) * complex_i_mul_real(x).pow(Nat.2 * m + Nat.1)
            complex_i_mul_real(x).pow(Nat.2 * m + Nat.1) = Complex.i * Complex.from_real(alternating_sign[Real](m) * x.pow(Nat.2 * m + Nat.1))
            complex_i_mul_real(x) = Complex.i * Complex.from_real(x)
            complex_i_mul_real(x) * complex_i_mul_real(x).pow(Nat.2 * m + Nat.1) =
                (Complex.i * Complex.from_real(x)) * (Complex.i * Complex.from_real(alternating_sign[Real](m) * x.pow(Nat.2 * m + Nat.1)))
            complex_i_mul_real_square(x, alternating_sign[Real](m) * x.pow(Nat.2 * m + Nat.1))
            (Complex.i * Complex.from_real(x)) * (Complex.i * Complex.from_real(alternating_sign[Real](m) * x.pow(Nat.2 * m + Nat.1))) =
                -Complex.from_real(x * (alternating_sign[Real](m) * x.pow(Nat.2 * m + Nat.1)))
            complex_i_mul_real(x).pow(Nat.2 * m.suc) = -Complex.from_real(x * (alternating_sign[Real](m) * x.pow(Nat.2 * m + Nat.1)))
            real_mul_comm(x, alternating_sign[Real](m) * x.pow(Nat.2 * m + Nat.1))
            x * (alternating_sign[Real](m) * x.pow(Nat.2 * m + Nat.1)) =
                (alternating_sign[Real](m) * x.pow(Nat.2 * m + Nat.1)) * x
            real_mul_assoc(alternating_sign[Real](m), x.pow(Nat.2 * m + Nat.1), x)
            (alternating_sign[Real](m) * x.pow(Nat.2 * m + Nat.1)) * x =
                alternating_sign[Real](m) * (x.pow(Nat.2 * m + Nat.1) * x)
            real_pow_suc(x, Nat.2 * m + Nat.1)
            x.pow((Nat.2 * m + Nat.1).suc) = x * x.pow(Nat.2 * m + Nat.1)
            real_mul_comm(x, x.pow(Nat.2 * m + Nat.1))
            x * x.pow(Nat.2 * m + Nat.1) = x.pow(Nat.2 * m + Nat.1) * x
            x.pow(Nat.2 * m + Nat.2) = x.pow(Nat.2 * m + Nat.1) * x
            x * (alternating_sign[Real](m) * x.pow(Nat.2 * m + Nat.1)) = alternating_sign[Real](m) * x.pow(Nat.2 * m + Nat.2)
            two_mul_suc(m)
            Nat.2 * m.suc = (Nat.2 * m).suc.suc
            Nat.2 * m + Nat.2 = (Nat.2 * m).suc.suc
            x.pow(Nat.2 * m + Nat.2) = x.pow(Nat.2 * m.suc)
            x * (alternating_sign[Real](m) * x.pow(Nat.2 * m + Nat.1)) = alternating_sign[Real](m) * x.pow(Nat.2 * m.suc)
            complex_i_mul_real(x).pow(Nat.2 * m.suc) = -Complex.from_real(alternating_sign[Real](m) * x.pow(Nat.2 * m.suc))
            from_real_neg(alternating_sign[Real](m) * x.pow(Nat.2 * m.suc))
            Complex.from_real(-(alternating_sign[Real](m) * x.pow(Nat.2 * m.suc))) =
                -Complex.from_real(alternating_sign[Real](m) * x.pow(Nat.2 * m.suc))
            complex_i_mul_real(x).pow(Nat.2 * m.suc) = Complex.from_real(-(alternating_sign[Real](m) * x.pow(Nat.2 * m.suc)))
            alternating_sign_suc[Real](m)
            alternating_sign[Real](m.suc) = -alternating_sign[Real](m)
            mul_neg_left(alternating_sign[Real](m), x.pow(Nat.2 * m.suc))
            (-alternating_sign[Real](m)) * x.pow(Nat.2 * m.suc) = -(alternating_sign[Real](m) * x.pow(Nat.2 * m.suc))
            alternating_sign[Real](m.suc) * x.pow(Nat.2 * m.suc) = -(alternating_sign[Real](m) * x.pow(Nat.2 * m.suc))
            complex_i_mul_real(x).pow(Nat.2 * m.suc) = Complex.from_real(alternating_sign[Real](m.suc) * x.pow(Nat.2 * m.suc))

            // odd part of p(m.suc):
            complex_pow_suc(complex_i_mul_real(x), Nat.2 * m.suc)
            complex_i_mul_real(x).pow((Nat.2 * m.suc).suc) = complex_i_mul_real(x) * complex_i_mul_real(x).pow(Nat.2 * m.suc)
            Nat.2 * m.suc + Nat.1 = (Nat.2 * m.suc).suc
            complex_i_mul_real(x).pow(Nat.2 * m.suc + Nat.1) = complex_i_mul_real(x) * complex_i_mul_real(x).pow(Nat.2 * m.suc)
            complex_i_mul_real(x).pow(Nat.2 * m.suc) = Complex.from_real(alternating_sign[Real](m.suc) * x.pow(Nat.2 * m.suc))
            complex_i_mul_real(x) = Complex.i * Complex.from_real(x)
            complex_i_mul_real(x) * complex_i_mul_real(x).pow(Nat.2 * m.suc) =
                (Complex.i * Complex.from_real(x)) * Complex.from_real(alternating_sign[Real](m.suc) * x.pow(Nat.2 * m.suc))
            mul_assoc(Complex.i, Complex.from_real(x), Complex.from_real(alternating_sign[Real](m.suc) * x.pow(Nat.2 * m.suc)))
            (Complex.i * Complex.from_real(x)) * Complex.from_real(alternating_sign[Real](m.suc) * x.pow(Nat.2 * m.suc)) =
                Complex.i * (Complex.from_real(x) * Complex.from_real(alternating_sign[Real](m.suc) * x.pow(Nat.2 * m.suc)))
            real_mul_lifts(x, alternating_sign[Real](m.suc) * x.pow(Nat.2 * m.suc))
            Complex.from_real(x * (alternating_sign[Real](m.suc) * x.pow(Nat.2 * m.suc))) =
                Complex.from_real(x) * Complex.from_real(alternating_sign[Real](m.suc) * x.pow(Nat.2 * m.suc))
            Complex.i * (Complex.from_real(x) * Complex.from_real(alternating_sign[Real](m.suc) * x.pow(Nat.2 * m.suc))) =
                Complex.i * Complex.from_real(x * (alternating_sign[Real](m.suc) * x.pow(Nat.2 * m.suc)))
            complex_i_mul_real(x).pow(Nat.2 * m.suc + Nat.1) =
                Complex.i * Complex.from_real(x * (alternating_sign[Real](m.suc) * x.pow(Nat.2 * m.suc)))
            real_mul_comm(x, alternating_sign[Real](m.suc) * x.pow(Nat.2 * m.suc))
            x * (alternating_sign[Real](m.suc) * x.pow(Nat.2 * m.suc)) =
                (alternating_sign[Real](m.suc) * x.pow(Nat.2 * m.suc)) * x
            real_mul_assoc(alternating_sign[Real](m.suc), x.pow(Nat.2 * m.suc), x)
            (alternating_sign[Real](m.suc) * x.pow(Nat.2 * m.suc)) * x =
                alternating_sign[Real](m.suc) * (x.pow(Nat.2 * m.suc) * x)
            real_pow_suc(x, Nat.2 * m.suc)
            x.pow((Nat.2 * m.suc).suc) = x * x.pow(Nat.2 * m.suc)
            real_mul_comm(x, x.pow(Nat.2 * m.suc))
            x * x.pow(Nat.2 * m.suc) = x.pow(Nat.2 * m.suc) * x
            x.pow(Nat.2 * m.suc + Nat.1) = x.pow(Nat.2 * m.suc) * x
            x * (alternating_sign[Real](m.suc) * x.pow(Nat.2 * m.suc)) =
                alternating_sign[Real](m.suc) * x.pow(Nat.2 * m.suc + Nat.1)
            complex_i_mul_real(x).pow(Nat.2 * m.suc + Nat.1) =
                Complex.i * Complex.from_real(alternating_sign[Real](m.suc) * x.pow(Nat.2 * m.suc + Nat.1))
            p(m.suc)
        }
    }
    p(Nat.0) and forall(m: Nat) { p(m) implies p(m.suc) }
    alt_induction(p)
    p(k)
}

/// The even powers of i·x are real: (i·x)^(2k) = (-1)^k x^(2k).
theorem complex_i_mul_real_pow_even(x: Real, k: Nat) {
    complex_i_mul_real(x).pow(Nat.2 * k) = Complex.from_real(alternating_sign[Real](k) * x.pow(Nat.2 * k))
} by {
    complex_i_mul_real_pow_parity(x, k)
}

/// The odd powers of i·x are imaginary: (i·x)^(2k+1) = i·(-1)^k x^(2k+1).
theorem complex_i_mul_real_pow_odd(x: Real, k: Nat) {
    complex_i_mul_real(x).pow(Nat.2 * k + Nat.1) = Complex.i * Complex.from_real(alternating_sign[Real](k) * x.pow(Nat.2 * k + Nat.1))
} by {
    complex_i_mul_real_pow_parity(x, k)
}

/// The real part of the even-indexed term of the complex exponential series at
/// i·x is the cosine term.
theorem complex_exp_term_i_mul_real_even_re(x: Real, k: Nat) {
    complex_exp_term(complex_i_mul_real(x), Nat.2 * k).re = cos_term(x, k)
} by {
    complex_exp_term(complex_i_mul_real(x), Nat.2 * k) =
        complex_i_mul_real(x).pow(Nat.2 * k) / Complex.from_real(Real.from_rat(Rat.from_nat((Nat.2 * k).factorial)))
    complex_i_mul_real_pow_even(x, k)
    complex_i_mul_real(x).pow(Nat.2 * k) = Complex.from_real(alternating_sign[Real](k) * x.pow(Nat.2 * k))
    complex_exp_term(complex_i_mul_real(x), Nat.2 * k) =
        Complex.from_real(alternating_sign[Real](k) * x.pow(Nat.2 * k)) / Complex.from_real(Real.from_rat(Rat.from_nat((Nat.2 * k).factorial)))
    div_from_real(alternating_sign[Real](k) * x.pow(Nat.2 * k), Real.from_rat(Rat.from_nat((Nat.2 * k).factorial)))
    Complex.from_real(alternating_sign[Real](k) * x.pow(Nat.2 * k)) / Complex.from_real(Real.from_rat(Rat.from_nat((Nat.2 * k).factorial))) =
        Complex.from_real((alternating_sign[Real](k) * x.pow(Nat.2 * k)) / Real.from_rat(Rat.from_nat((Nat.2 * k).factorial)))
    complex_exp_term(complex_i_mul_real(x), Nat.2 * k) =
        Complex.from_real((alternating_sign[Real](k) * x.pow(Nat.2 * k)) / Real.from_rat(Rat.from_nat((Nat.2 * k).factorial)))
    complex_exp_term(complex_i_mul_real(x), Nat.2 * k).re =
        Complex.from_real((alternating_sign[Real](k) * x.pow(Nat.2 * k)) / Real.from_rat(Rat.from_nat((Nat.2 * k).factorial))).re
    re_from_real((alternating_sign[Real](k) * x.pow(Nat.2 * k)) / Real.from_rat(Rat.from_nat((Nat.2 * k).factorial)))
    Complex.from_real((alternating_sign[Real](k) * x.pow(Nat.2 * k)) / Real.from_rat(Rat.from_nat((Nat.2 * k).factorial))).re =
        (alternating_sign[Real](k) * x.pow(Nat.2 * k)) / Real.from_rat(Rat.from_nat((Nat.2 * k).factorial))
    complex_exp_term(complex_i_mul_real(x), Nat.2 * k).re =
        (alternating_sign[Real](k) * x.pow(Nat.2 * k)) / Real.from_rat(Rat.from_nat((Nat.2 * k).factorial))
    cos_term(x, k) = alternating_sign[Real](k) * x.pow(Nat.2 * k) / Real.from_rat(Rat.from_nat((Nat.2 * k).factorial))
    complex_exp_term(complex_i_mul_real(x), Nat.2 * k).re = cos_term(x, k)
}

/// The imaginary part of the even-indexed term of the complex exponential
/// series at i·x vanishes.
theorem complex_exp_term_i_mul_real_even_im(x: Real, k: Nat) {
    complex_exp_term(complex_i_mul_real(x), Nat.2 * k).im = Real.0
} by {
    complex_exp_term(complex_i_mul_real(x), Nat.2 * k) =
        complex_i_mul_real(x).pow(Nat.2 * k) / Complex.from_real(Real.from_rat(Rat.from_nat((Nat.2 * k).factorial)))
    complex_i_mul_real_pow_even(x, k)
    complex_i_mul_real(x).pow(Nat.2 * k) = Complex.from_real(alternating_sign[Real](k) * x.pow(Nat.2 * k))
    complex_exp_term(complex_i_mul_real(x), Nat.2 * k) =
        Complex.from_real(alternating_sign[Real](k) * x.pow(Nat.2 * k)) / Complex.from_real(Real.from_rat(Rat.from_nat((Nat.2 * k).factorial)))
    div_from_real(alternating_sign[Real](k) * x.pow(Nat.2 * k), Real.from_rat(Rat.from_nat((Nat.2 * k).factorial)))
    Complex.from_real(alternating_sign[Real](k) * x.pow(Nat.2 * k)) / Complex.from_real(Real.from_rat(Rat.from_nat((Nat.2 * k).factorial))) =
        Complex.from_real((alternating_sign[Real](k) * x.pow(Nat.2 * k)) / Real.from_rat(Rat.from_nat((Nat.2 * k).factorial)))
    complex_exp_term(complex_i_mul_real(x), Nat.2 * k).im =
        Complex.from_real((alternating_sign[Real](k) * x.pow(Nat.2 * k)) / Real.from_rat(Rat.from_nat((Nat.2 * k).factorial))).im
    im_from_real((alternating_sign[Real](k) * x.pow(Nat.2 * k)) / Real.from_rat(Rat.from_nat((Nat.2 * k).factorial)))
    Complex.from_real((alternating_sign[Real](k) * x.pow(Nat.2 * k)) / Real.from_rat(Rat.from_nat((Nat.2 * k).factorial))).im = Real.0
    complex_exp_term(complex_i_mul_real(x), Nat.2 * k).im = Real.0
}

/// The real part of the odd-indexed term of the complex exponential series at
/// i·x vanishes.
theorem complex_exp_term_i_mul_real_odd_re(x: Real, k: Nat) {
    complex_exp_term(complex_i_mul_real(x), Nat.2 * k + Nat.1).re = Real.0
} by {
    complex_exp_term(complex_i_mul_real(x), Nat.2 * k + Nat.1) =
        complex_i_mul_real(x).pow(Nat.2 * k + Nat.1) / Complex.from_real(Real.from_rat(Rat.from_nat((Nat.2 * k + Nat.1).factorial)))
    complex_i_mul_real_pow_odd(x, k)
    complex_i_mul_real(x).pow(Nat.2 * k + Nat.1) = Complex.i * Complex.from_real(alternating_sign[Real](k) * x.pow(Nat.2 * k + Nat.1))
    complex_exp_term(complex_i_mul_real(x), Nat.2 * k + Nat.1) =
        (Complex.i * Complex.from_real(alternating_sign[Real](k) * x.pow(Nat.2 * k + Nat.1))) /
        Complex.from_real(Real.from_rat(Rat.from_nat((Nat.2 * k + Nat.1).factorial)))
    mul_assoc(Complex.i, Complex.from_real(alternating_sign[Real](k) * x.pow(Nat.2 * k + Nat.1)),
        Complex.from_real(Real.from_rat(Rat.from_nat((Nat.2 * k + Nat.1).factorial))).inverse)
    (Complex.i * Complex.from_real(alternating_sign[Real](k) * x.pow(Nat.2 * k + Nat.1))) *
        Complex.from_real(Real.from_rat(Rat.from_nat((Nat.2 * k + Nat.1).factorial))).inverse =
        Complex.i * (Complex.from_real(alternating_sign[Real](k) * x.pow(Nat.2 * k + Nat.1)) *
            Complex.from_real(Real.from_rat(Rat.from_nat((Nat.2 * k + Nat.1).factorial))).inverse)
    (Complex.i * Complex.from_real(alternating_sign[Real](k) * x.pow(Nat.2 * k + Nat.1))) /
        Complex.from_real(Real.from_rat(Rat.from_nat((Nat.2 * k + Nat.1).factorial))) =
        Complex.i * (Complex.from_real(alternating_sign[Real](k) * x.pow(Nat.2 * k + Nat.1)) /
            Complex.from_real(Real.from_rat(Rat.from_nat((Nat.2 * k + Nat.1).factorial))))
    div_from_real(alternating_sign[Real](k) * x.pow(Nat.2 * k + Nat.1), Real.from_rat(Rat.from_nat((Nat.2 * k + Nat.1).factorial)))
    Complex.from_real(alternating_sign[Real](k) * x.pow(Nat.2 * k + Nat.1)) / Complex.from_real(Real.from_rat(Rat.from_nat((Nat.2 * k + Nat.1).factorial))) =
        Complex.from_real((alternating_sign[Real](k) * x.pow(Nat.2 * k + Nat.1)) / Real.from_rat(Rat.from_nat((Nat.2 * k + Nat.1).factorial)))
    complex_exp_term(complex_i_mul_real(x), Nat.2 * k + Nat.1) =
        Complex.i * Complex.from_real((alternating_sign[Real](k) * x.pow(Nat.2 * k + Nat.1)) / Real.from_rat(Rat.from_nat((Nat.2 * k + Nat.1).factorial)))
    mul_from_real(Complex.i, (alternating_sign[Real](k) * x.pow(Nat.2 * k + Nat.1)) / Real.from_rat(Rat.from_nat((Nat.2 * k + Nat.1).factorial)))
    Complex.i * Complex.from_real((alternating_sign[Real](k) * x.pow(Nat.2 * k + Nat.1)) / Real.from_rat(Rat.from_nat((Nat.2 * k + Nat.1).factorial))) =
        Complex.new(Complex.i.re * ((alternating_sign[Real](k) * x.pow(Nat.2 * k + Nat.1)) / Real.from_rat(Rat.from_nat((Nat.2 * k + Nat.1).factorial))),
            Complex.i.im * ((alternating_sign[Real](k) * x.pow(Nat.2 * k + Nat.1)) / Real.from_rat(Rat.from_nat((Nat.2 * k + Nat.1).factorial))))
    complex_exp_term(complex_i_mul_real(x), Nat.2 * k + Nat.1) =
        Complex.new(Complex.i.re * ((alternating_sign[Real](k) * x.pow(Nat.2 * k + Nat.1)) / Real.from_rat(Rat.from_nat((Nat.2 * k + Nat.1).factorial))),
            Complex.i.im * ((alternating_sign[Real](k) * x.pow(Nat.2 * k + Nat.1)) / Real.from_rat(Rat.from_nat((Nat.2 * k + Nat.1).factorial))))
    complex_exp_term(complex_i_mul_real(x), Nat.2 * k + Nat.1).re =
        Complex.new(Complex.i.re * ((alternating_sign[Real](k) * x.pow(Nat.2 * k + Nat.1)) / Real.from_rat(Rat.from_nat((Nat.2 * k + Nat.1).factorial))),
            Complex.i.im * ((alternating_sign[Real](k) * x.pow(Nat.2 * k + Nat.1)) / Real.from_rat(Rat.from_nat((Nat.2 * k + Nat.1).factorial)))).re
    Complex.new(Complex.i.re * ((alternating_sign[Real](k) * x.pow(Nat.2 * k + Nat.1)) / Real.from_rat(Rat.from_nat((Nat.2 * k + Nat.1).factorial))),
        Complex.i.im * ((alternating_sign[Real](k) * x.pow(Nat.2 * k + Nat.1)) / Real.from_rat(Rat.from_nat((Nat.2 * k + Nat.1).factorial)))).re =
        Complex.i.re * ((alternating_sign[Real](k) * x.pow(Nat.2 * k + Nat.1)) / Real.from_rat(Rat.from_nat((Nat.2 * k + Nat.1).factorial)))
    complex_exp_term(complex_i_mul_real(x), Nat.2 * k + Nat.1).re =
        Complex.i.re * ((alternating_sign[Real](k) * x.pow(Nat.2 * k + Nat.1)) / Real.from_rat(Rat.from_nat((Nat.2 * k + Nat.1).factorial)))
    re_i
    Complex.i.re = Real.0
    Real.0 * ((alternating_sign[Real](k) * x.pow(Nat.2 * k + Nat.1)) / Real.from_rat(Rat.from_nat((Nat.2 * k + Nat.1).factorial))) = Real.0
    complex_exp_term(complex_i_mul_real(x), Nat.2 * k + Nat.1).re = Real.0
}

/// The imaginary part of the odd-indexed term of the complex exponential series
/// at i·x is the sine term.
theorem complex_exp_term_i_mul_real_odd_im(x: Real, k: Nat) {
    complex_exp_term(complex_i_mul_real(x), Nat.2 * k + Nat.1).im = sin_term(x, k)
} by {
    complex_exp_term(complex_i_mul_real(x), Nat.2 * k + Nat.1) =
        complex_i_mul_real(x).pow(Nat.2 * k + Nat.1) / Complex.from_real(Real.from_rat(Rat.from_nat((Nat.2 * k + Nat.1).factorial)))
    complex_i_mul_real_pow_odd(x, k)
    complex_i_mul_real(x).pow(Nat.2 * k + Nat.1) = Complex.i * Complex.from_real(alternating_sign[Real](k) * x.pow(Nat.2 * k + Nat.1))
    complex_exp_term(complex_i_mul_real(x), Nat.2 * k + Nat.1) =
        (Complex.i * Complex.from_real(alternating_sign[Real](k) * x.pow(Nat.2 * k + Nat.1))) /
        Complex.from_real(Real.from_rat(Rat.from_nat((Nat.2 * k + Nat.1).factorial)))
    mul_assoc(Complex.i, Complex.from_real(alternating_sign[Real](k) * x.pow(Nat.2 * k + Nat.1)),
        Complex.from_real(Real.from_rat(Rat.from_nat((Nat.2 * k + Nat.1).factorial))).inverse)
    (Complex.i * Complex.from_real(alternating_sign[Real](k) * x.pow(Nat.2 * k + Nat.1))) *
        Complex.from_real(Real.from_rat(Rat.from_nat((Nat.2 * k + Nat.1).factorial))).inverse =
        Complex.i * (Complex.from_real(alternating_sign[Real](k) * x.pow(Nat.2 * k + Nat.1)) *
            Complex.from_real(Real.from_rat(Rat.from_nat((Nat.2 * k + Nat.1).factorial))).inverse)
    (Complex.i * Complex.from_real(alternating_sign[Real](k) * x.pow(Nat.2 * k + Nat.1))) /
        Complex.from_real(Real.from_rat(Rat.from_nat((Nat.2 * k + Nat.1).factorial))) =
        Complex.i * (Complex.from_real(alternating_sign[Real](k) * x.pow(Nat.2 * k + Nat.1)) /
            Complex.from_real(Real.from_rat(Rat.from_nat((Nat.2 * k + Nat.1).factorial))))
    div_from_real(alternating_sign[Real](k) * x.pow(Nat.2 * k + Nat.1), Real.from_rat(Rat.from_nat((Nat.2 * k + Nat.1).factorial)))
    Complex.from_real(alternating_sign[Real](k) * x.pow(Nat.2 * k + Nat.1)) / Complex.from_real(Real.from_rat(Rat.from_nat((Nat.2 * k + Nat.1).factorial))) =
        Complex.from_real((alternating_sign[Real](k) * x.pow(Nat.2 * k + Nat.1)) / Real.from_rat(Rat.from_nat((Nat.2 * k + Nat.1).factorial)))
    complex_exp_term(complex_i_mul_real(x), Nat.2 * k + Nat.1) =
        Complex.i * Complex.from_real((alternating_sign[Real](k) * x.pow(Nat.2 * k + Nat.1)) / Real.from_rat(Rat.from_nat((Nat.2 * k + Nat.1).factorial)))
    mul_from_real(Complex.i, (alternating_sign[Real](k) * x.pow(Nat.2 * k + Nat.1)) / Real.from_rat(Rat.from_nat((Nat.2 * k + Nat.1).factorial)))
    Complex.i * Complex.from_real((alternating_sign[Real](k) * x.pow(Nat.2 * k + Nat.1)) / Real.from_rat(Rat.from_nat((Nat.2 * k + Nat.1).factorial))) =
        Complex.new(Complex.i.re * ((alternating_sign[Real](k) * x.pow(Nat.2 * k + Nat.1)) / Real.from_rat(Rat.from_nat((Nat.2 * k + Nat.1).factorial))),
            Complex.i.im * ((alternating_sign[Real](k) * x.pow(Nat.2 * k + Nat.1)) / Real.from_rat(Rat.from_nat((Nat.2 * k + Nat.1).factorial))))
    complex_exp_term(complex_i_mul_real(x), Nat.2 * k + Nat.1) =
        Complex.new(Complex.i.re * ((alternating_sign[Real](k) * x.pow(Nat.2 * k + Nat.1)) / Real.from_rat(Rat.from_nat((Nat.2 * k + Nat.1).factorial))),
            Complex.i.im * ((alternating_sign[Real](k) * x.pow(Nat.2 * k + Nat.1)) / Real.from_rat(Rat.from_nat((Nat.2 * k + Nat.1).factorial))))
    complex_exp_term(complex_i_mul_real(x), Nat.2 * k + Nat.1).im =
        Complex.new(Complex.i.re * ((alternating_sign[Real](k) * x.pow(Nat.2 * k + Nat.1)) / Real.from_rat(Rat.from_nat((Nat.2 * k + Nat.1).factorial))),
            Complex.i.im * ((alternating_sign[Real](k) * x.pow(Nat.2 * k + Nat.1)) / Real.from_rat(Rat.from_nat((Nat.2 * k + Nat.1).factorial)))).im
    Complex.new(Complex.i.re * ((alternating_sign[Real](k) * x.pow(Nat.2 * k + Nat.1)) / Real.from_rat(Rat.from_nat((Nat.2 * k + Nat.1).factorial))),
        Complex.i.im * ((alternating_sign[Real](k) * x.pow(Nat.2 * k + Nat.1)) / Real.from_rat(Rat.from_nat((Nat.2 * k + Nat.1).factorial)))).im =
        Complex.i.im * ((alternating_sign[Real](k) * x.pow(Nat.2 * k + Nat.1)) / Real.from_rat(Rat.from_nat((Nat.2 * k + Nat.1).factorial)))
    complex_exp_term(complex_i_mul_real(x), Nat.2 * k + Nat.1).im =
        Complex.i.im * ((alternating_sign[Real](k) * x.pow(Nat.2 * k + Nat.1)) / Real.from_rat(Rat.from_nat((Nat.2 * k + Nat.1).factorial)))
    im_i
    Complex.i.im = Real.1
    Real.1 * ((alternating_sign[Real](k) * x.pow(Nat.2 * k + Nat.1)) / Real.from_rat(Rat.from_nat((Nat.2 * k + Nat.1).factorial))) =
        (alternating_sign[Real](k) * x.pow(Nat.2 * k + Nat.1)) / Real.from_rat(Rat.from_nat((Nat.2 * k + Nat.1).factorial))
    complex_exp_term(complex_i_mul_real(x), Nat.2 * k + Nat.1).im =
        (alternating_sign[Real](k) * x.pow(Nat.2 * k + Nat.1)) / Real.from_rat(Rat.from_nat((Nat.2 * k + Nat.1).factorial))
    sin_term(x, k) = alternating_sign[Real](k) * x.pow(Nat.2 * k + Nat.1) / Real.from_rat(Rat.from_nat((Nat.2 * k + Nat.1).factorial))
    complex_exp_term(complex_i_mul_real(x), Nat.2 * k + Nat.1).im = sin_term(x, k)
}

/// The real parts of the first 2k terms of the complex exponential series at
/// i·x are the first k cosine terms.
theorem complex_exp_i_mul_real_partial_re(x: Real, k: Nat) {
    partial(re_seq(complex_exp_term(complex_i_mul_real(x))), Nat.2 * k) = partial(cos_term(x), k)
} by {
    define p(m: Nat) -> Bool {
        partial(re_seq(complex_exp_term(complex_i_mul_real(x))), Nat.2 * m) = partial(cos_term(x), m)
    }
    partial(re_seq(complex_exp_term(complex_i_mul_real(x))), Nat.0) = Real.0
    partial(cos_term(x), Nat.0) = Real.0
    p(Nat.0)
    forall(m: Nat) {
        if p(m) {
            partial(re_seq(complex_exp_term(complex_i_mul_real(x))), Nat.2 * m) = partial(cos_term(x), m)
            two_mul_suc(m)
            Nat.2 * m.suc = (Nat.2 * m).suc.suc
            partial_suc(re_seq(complex_exp_term(complex_i_mul_real(x))), Nat.2 * m + Nat.1)
            partial(re_seq(complex_exp_term(complex_i_mul_real(x))), Nat.2 * m.suc) =
                partial(re_seq(complex_exp_term(complex_i_mul_real(x))), Nat.2 * m + Nat.1) +
                re_seq(complex_exp_term(complex_i_mul_real(x)), Nat.2 * m + Nat.1)
            Nat.2 * m.suc = (Nat.2 * m).suc.suc
            partial_suc(re_seq(complex_exp_term(complex_i_mul_real(x))), Nat.2 * m)
            partial(re_seq(complex_exp_term(complex_i_mul_real(x))), Nat.2 * m + Nat.1) =
                partial(re_seq(complex_exp_term(complex_i_mul_real(x))), Nat.2 * m) +
                re_seq(complex_exp_term(complex_i_mul_real(x)), Nat.2 * m)
            partial(re_seq(complex_exp_term(complex_i_mul_real(x))), Nat.2 * m.suc) =
                partial(re_seq(complex_exp_term(complex_i_mul_real(x))), Nat.2 * m) +
                re_seq(complex_exp_term(complex_i_mul_real(x)), Nat.2 * m) +
                re_seq(complex_exp_term(complex_i_mul_real(x)), Nat.2 * m + Nat.1)
            re_seq(complex_exp_term(complex_i_mul_real(x)), Nat.2 * m) = complex_exp_term(complex_i_mul_real(x), Nat.2 * m).re
            complex_exp_term_i_mul_real_even_re(x, m)
            complex_exp_term(complex_i_mul_real(x), Nat.2 * m).re = cos_term(x, m)
            re_seq(complex_exp_term(complex_i_mul_real(x)), Nat.2 * m) = cos_term(x, m)
            re_seq(complex_exp_term(complex_i_mul_real(x)), Nat.2 * m + Nat.1) = complex_exp_term(complex_i_mul_real(x), Nat.2 * m + Nat.1).re
            complex_exp_term_i_mul_real_odd_re(x, m)
            complex_exp_term(complex_i_mul_real(x), Nat.2 * m + Nat.1).re = Real.0
            re_seq(complex_exp_term(complex_i_mul_real(x)), Nat.2 * m + Nat.1) = Real.0
            partial(re_seq(complex_exp_term(complex_i_mul_real(x))), Nat.2 * m.suc) =
                partial(cos_term(x), m) + cos_term(x, m) + Real.0
            partial_suc(cos_term(x), m)
            partial(cos_term(x), m.suc) = partial(cos_term(x), m) + cos_term(x, m)
            partial(re_seq(complex_exp_term(complex_i_mul_real(x))), Nat.2 * m.suc) = partial(cos_term(x), m.suc)
            p(m.suc)
        }
    }
    p(Nat.0) and forall(m: Nat) { p(m) implies p(m.suc) }
    alt_induction(p)
    p(k)
}

/// The imaginary parts of the first 2k terms of the complex exponential series
/// at i·x are the first k sine terms.
theorem complex_exp_i_mul_real_partial_im(x: Real, k: Nat) {
    partial(im_seq(complex_exp_term(complex_i_mul_real(x))), Nat.2 * k) = partial(sin_term(x), k)
} by {
    define p(m: Nat) -> Bool {
        partial(im_seq(complex_exp_term(complex_i_mul_real(x))), Nat.2 * m) = partial(sin_term(x), m)
    }
    partial(im_seq(complex_exp_term(complex_i_mul_real(x))), Nat.0) = Real.0
    partial(sin_term(x), Nat.0) = Real.0
    p(Nat.0)
    forall(m: Nat) {
        if p(m) {
            partial(im_seq(complex_exp_term(complex_i_mul_real(x))), Nat.2 * m) = partial(sin_term(x), m)
            two_mul_suc(m)
            Nat.2 * m.suc = (Nat.2 * m).suc.suc
            partial_suc(im_seq(complex_exp_term(complex_i_mul_real(x))), Nat.2 * m + Nat.1)
            partial(im_seq(complex_exp_term(complex_i_mul_real(x))), Nat.2 * m.suc) =
                partial(im_seq(complex_exp_term(complex_i_mul_real(x))), Nat.2 * m + Nat.1) +
                im_seq(complex_exp_term(complex_i_mul_real(x)), Nat.2 * m + Nat.1)
            Nat.2 * m.suc = (Nat.2 * m).suc.suc
            partial_suc(im_seq(complex_exp_term(complex_i_mul_real(x))), Nat.2 * m)
            partial(im_seq(complex_exp_term(complex_i_mul_real(x))), Nat.2 * m + Nat.1) =
                partial(im_seq(complex_exp_term(complex_i_mul_real(x))), Nat.2 * m) +
                im_seq(complex_exp_term(complex_i_mul_real(x)), Nat.2 * m)
            partial(im_seq(complex_exp_term(complex_i_mul_real(x))), Nat.2 * m.suc) =
                partial(im_seq(complex_exp_term(complex_i_mul_real(x))), Nat.2 * m) +
                im_seq(complex_exp_term(complex_i_mul_real(x)), Nat.2 * m) +
                im_seq(complex_exp_term(complex_i_mul_real(x)), Nat.2 * m + Nat.1)
            im_seq(complex_exp_term(complex_i_mul_real(x)), Nat.2 * m) = complex_exp_term(complex_i_mul_real(x), Nat.2 * m).im
            complex_exp_term_i_mul_real_even_im(x, m)
            complex_exp_term(complex_i_mul_real(x), Nat.2 * m).im = Real.0
            im_seq(complex_exp_term(complex_i_mul_real(x)), Nat.2 * m) = Real.0
            im_seq(complex_exp_term(complex_i_mul_real(x)), Nat.2 * m + Nat.1) = complex_exp_term(complex_i_mul_real(x), Nat.2 * m + Nat.1).im
            complex_exp_term_i_mul_real_odd_im(x, m)
            complex_exp_term(complex_i_mul_real(x), Nat.2 * m + Nat.1).im = sin_term(x, m)
            im_seq(complex_exp_term(complex_i_mul_real(x)), Nat.2 * m + Nat.1) = sin_term(x, m)
            partial(im_seq(complex_exp_term(complex_i_mul_real(x))), Nat.2 * m.suc) =
                partial(sin_term(x), m) + Real.0 + sin_term(x, m)
            partial_suc(sin_term(x), m)
            partial(sin_term(x), m.suc) = partial(sin_term(x), m) + sin_term(x, m)
            partial(im_seq(complex_exp_term(complex_i_mul_real(x))), Nat.2 * m.suc) = partial(sin_term(x), m.suc)
            p(m.suc)
        }
    }
    p(Nat.0) and forall(m: Nat) { p(m) implies p(m.suc) }
    alt_induction(p)
    p(k)
}

/// The real part of the exponential of i·x is the cosine of x.
theorem complex_exp_i_mul_real_re(x: Real) {
    complex_exp(complex_i_mul_real(x)).re = x.cos
} by {
    complex_exp(complex_i_mul_real(x)) = complex_limit(partial(complex_exp_term(complex_i_mul_real(x))))
    complex_exp_series_converges(complex_i_mul_real(x))
    complex_converges(partial(complex_exp_term(complex_i_mul_real(x))))
    complex_limit_re(partial(complex_exp_term(complex_i_mul_real(x))))
    limit(re_seq(partial(complex_exp_term(complex_i_mul_real(x))))) =
        complex_limit(partial(complex_exp_term(complex_i_mul_real(x)))).re
    complex_exp(complex_i_mul_real(x)).re = complex_limit(partial(complex_exp_term(complex_i_mul_real(x)))).re
    complex_exp(complex_i_mul_real(x)).re = limit(re_seq(partial(complex_exp_term(complex_i_mul_real(x)))))
    forall(n: Nat) {
        partial_re_seq(complex_exp_term(complex_i_mul_real(x)), n)
        re_seq(partial(complex_exp_term(complex_i_mul_real(x))), n) = partial(re_seq(complex_exp_term(complex_i_mul_real(x))), n)
    }
    re_seq(partial(complex_exp_term(complex_i_mul_real(x)))) = partial(re_seq(complex_exp_term(complex_i_mul_real(x))))
    complex_exp(complex_i_mul_real(x)).re = limit(partial(re_seq(complex_exp_term(complex_i_mul_real(x)))))

    complex_exp_series_re_converges(complex_i_mul_real(x))
    converges(partial(re_seq(complex_exp_term(complex_i_mul_real(x)))))
    converges_subsequence_double(partial(re_seq(complex_exp_term(complex_i_mul_real(x)))))
    converges_to(subsequence(partial(re_seq(complex_exp_term(complex_i_mul_real(x)))), Nat.2.mul),
        limit(partial(re_seq(complex_exp_term(complex_i_mul_real(x))))))
    forall(k: Nat) {
        complex_exp_i_mul_real_partial_re(x, k)
        partial(re_seq(complex_exp_term(complex_i_mul_real(x))), Nat.2 * k) = partial(cos_term(x), k)
        subsequence(partial(re_seq(complex_exp_term(complex_i_mul_real(x)))), Nat.2.mul, k) =
            partial(re_seq(complex_exp_term(complex_i_mul_real(x))), Nat.2 * k)
        subsequence(partial(re_seq(complex_exp_term(complex_i_mul_real(x)))), Nat.2.mul, k) = partial(cos_term(x), k)
    }
    subsequence(partial(re_seq(complex_exp_term(complex_i_mul_real(x)))), Nat.2.mul) = partial(cos_term(x))
    cos_term_abs_converges(x)
    absolutely_converges(cos_term(x))
    absolutely_converges_imp_converges(cos_term(x))
    converges(partial(cos_term(x)))
    converges_imp_converges_to(partial(cos_term(x)))
    converges_to(partial(cos_term(x)), limit(partial(cos_term(x))))
    x.cos = limit(partial(cos_term(x)))
    converges_to(partial(cos_term(x)), x.cos)
    converges_to(subsequence(partial(re_seq(complex_exp_term(complex_i_mul_real(x)))), Nat.2.mul), x.cos)
    converges_to_unique(subsequence(partial(re_seq(complex_exp_term(complex_i_mul_real(x)))), Nat.2.mul),
        limit(partial(re_seq(complex_exp_term(complex_i_mul_real(x))))), x.cos)
    limit(partial(re_seq(complex_exp_term(complex_i_mul_real(x))))) = x.cos
    complex_exp(complex_i_mul_real(x)).re = x.cos
}

/// The imaginary part of the exponential of i·x is the sine of x.
theorem complex_exp_i_mul_real_im(x: Real) {
    complex_exp(complex_i_mul_real(x)).im = x.sin
} by {
    complex_exp(complex_i_mul_real(x)) = complex_limit(partial(complex_exp_term(complex_i_mul_real(x))))
    complex_exp_series_converges(complex_i_mul_real(x))
    complex_converges(partial(complex_exp_term(complex_i_mul_real(x))))
    complex_limit_im(partial(complex_exp_term(complex_i_mul_real(x))))
    limit(im_seq(partial(complex_exp_term(complex_i_mul_real(x))))) =
        complex_limit(partial(complex_exp_term(complex_i_mul_real(x)))).im
    complex_exp(complex_i_mul_real(x)).im = complex_limit(partial(complex_exp_term(complex_i_mul_real(x)))).im
    complex_exp(complex_i_mul_real(x)).im = limit(im_seq(partial(complex_exp_term(complex_i_mul_real(x)))))
    forall(n: Nat) {
        partial_im_seq(complex_exp_term(complex_i_mul_real(x)), n)
        im_seq(partial(complex_exp_term(complex_i_mul_real(x))), n) = partial(im_seq(complex_exp_term(complex_i_mul_real(x))), n)
    }
    im_seq(partial(complex_exp_term(complex_i_mul_real(x)))) = partial(im_seq(complex_exp_term(complex_i_mul_real(x))))
    complex_exp(complex_i_mul_real(x)).im = limit(partial(im_seq(complex_exp_term(complex_i_mul_real(x)))))

    complex_exp_series_im_converges(complex_i_mul_real(x))
    converges(partial(im_seq(complex_exp_term(complex_i_mul_real(x)))))
    converges_subsequence_double(partial(im_seq(complex_exp_term(complex_i_mul_real(x)))))
    converges_to(subsequence(partial(im_seq(complex_exp_term(complex_i_mul_real(x)))), Nat.2.mul),
        limit(partial(im_seq(complex_exp_term(complex_i_mul_real(x))))))
    forall(k: Nat) {
        complex_exp_i_mul_real_partial_im(x, k)
        partial(im_seq(complex_exp_term(complex_i_mul_real(x))), Nat.2 * k) = partial(sin_term(x), k)
        subsequence(partial(im_seq(complex_exp_term(complex_i_mul_real(x)))), Nat.2.mul, k) =
            partial(im_seq(complex_exp_term(complex_i_mul_real(x))), Nat.2 * k)
        subsequence(partial(im_seq(complex_exp_term(complex_i_mul_real(x)))), Nat.2.mul, k) = partial(sin_term(x), k)
    }
    subsequence(partial(im_seq(complex_exp_term(complex_i_mul_real(x)))), Nat.2.mul) = partial(sin_term(x))
    sin_term_abs_converges(x)
    absolutely_converges(sin_term(x))
    absolutely_converges_imp_converges(sin_term(x))
    converges(partial(sin_term(x)))
    converges_imp_converges_to(partial(sin_term(x)))
    converges_to(partial(sin_term(x)), limit(partial(sin_term(x))))
    x.sin = limit(partial(sin_term(x)))
    converges_to(partial(sin_term(x)), x.sin)
    converges_to(subsequence(partial(im_seq(complex_exp_term(complex_i_mul_real(x)))), Nat.2.mul), x.sin)
    converges_to_unique(subsequence(partial(im_seq(complex_exp_term(complex_i_mul_real(x)))), Nat.2.mul),
        limit(partial(im_seq(complex_exp_term(complex_i_mul_real(x))))), x.sin)
    limit(partial(im_seq(complex_exp_term(complex_i_mul_real(x))))) = x.sin
    complex_exp(complex_i_mul_real(x)).im = x.sin
}

/// The exponential of i·x is x.cos + i·x.sin.
theorem complex_exp_i_mul_real_eq(x: Real) {
    complex_exp(complex_i_mul_real(x)) = Complex.new(x.cos, x.sin)
} by {
    complex_exp_i_mul_real_re(x)
    complex_exp(complex_i_mul_real(x)).re = x.cos
    complex_exp_i_mul_real_im(x)
    complex_exp(complex_i_mul_real(x)).im = x.sin
    Complex.new(x.cos, x.sin).re = x.cos
    Complex.new(x.cos, x.sin).im = x.sin
    eq_by_components(complex_exp(complex_i_mul_real(x)), Complex.new(x.cos, x.sin))
    complex_exp(complex_i_mul_real(x)) = Complex.new(x.cos, x.sin)
}

/// Euler's identity: (i·pi).exp = -1.
theorem euler_identity {
    complex_exp(Complex.i * Complex.from_real(pi)) = -Complex.1
} by {
    complex_i_mul_real(pi) = Complex.i * Complex.from_real(pi)
    complex_exp(complex_i_mul_real(pi)) = complex_exp(Complex.i * Complex.from_real(pi))
    complex_exp_i_mul_real_eq(pi)
    complex_exp(complex_i_mul_real(pi)) = Complex.new(pi.cos, pi.sin)
    cos_pi_neg_one
    pi.cos = -Real.1
    sin_pi_zero
    pi.sin = Real.0
    Complex.new(pi.cos, pi.sin) = Complex.new(-Real.1, Real.0)
    complex_neg_one_new
    Complex.new(-Real.1, Real.0) = -Complex.1
    complex_exp(Complex.i * Complex.from_real(pi)) = Complex.new(-Real.1, Real.0)
    complex_exp(Complex.i * Complex.from_real(pi)) = -Complex.1
}

/// Euler's identity in additive form: (i·pi).exp + 1 = 0.
theorem euler_identity_plus_one {
    complex_exp(Complex.i * Complex.from_real(pi)) + Complex.1 = Complex.0
} by {
    euler_identity
    complex_exp(Complex.i * Complex.from_real(pi)) = -Complex.1
    -Complex.1 + Complex.1 = Complex.0
    complex_exp(Complex.i * Complex.from_real(pi)) + Complex.1 = Complex.0
}

// ---------------------------------------------------------------------------
// The complex exponential is complete: (z + w).exp = z.exp * w.exp and
// Euler's identity (i·pi).exp = -1 are both proved above.
// ---------------------------------------------------------------------------
