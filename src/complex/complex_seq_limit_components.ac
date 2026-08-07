from nat import Nat
from real import limit
from complex.complex import Complex, re_add, im_add, re_sub, im_sub, re_mul, im_mul,
    neg_re, neg_im, re_conj, im_conj
from complex.complex_seq import re_seq, im_seq, complex_converges, complex_limit,
    complex_limit_re, complex_limit_im,
    neg_seq, conj_seq, complex_add_seq, complex_sub_seq, complex_mul_seq,
    neg_seq_converges, conj_seq_converges, complex_add_seq_converges,
    complex_sub_seq_converges, complex_mul_seq_converges,
    complex_limit_of_neg_seq, complex_limit_of_conj_seq, complex_limit_of_add_seq,
    complex_limit_of_sub_seq, complex_limit_of_mul_seq

/// A convergent complex sequence has the expected componentwise-limit constructor value.
theorem complex_limit_eq_component_limits(z: Nat -> Complex) {
    complex_converges(z) implies complex_limit(z) = Complex.new(limit(re_seq(z)), limit(im_seq(z)))
} by {
    if complex_converges(z) {
        complex_limit_re(z)
        complex_limit_im(z)
        complex_limit(z) = Complex.new(limit(re_seq(z)), limit(im_seq(z)))
    }
}

/// The real component limit of the negated sequence is the negated real component limit.
theorem complex_limit_re_of_neg_seq(z: Nat -> Complex) {
    complex_converges(z) implies limit(re_seq(neg_seq(z))) = -limit(re_seq(z))
} by {
    if complex_converges(z) {
        neg_seq_converges(z)
        complex_limit_re(neg_seq(z))
        complex_limit_of_neg_seq(z)
        neg_re(complex_limit(z))
        complex_limit_re(z)
        limit(re_seq(neg_seq(z))) = -limit(re_seq(z))
    }
}

/// The imaginary component limit of the negated sequence is the negated imaginary component limit.
theorem complex_limit_im_of_neg_seq(z: Nat -> Complex) {
    complex_converges(z) implies limit(im_seq(neg_seq(z))) = -limit(im_seq(z))
} by {
    if complex_converges(z) {
        neg_seq_converges(z)
        complex_limit_im(neg_seq(z))
        complex_limit_of_neg_seq(z)
        neg_im(complex_limit(z))
        complex_limit_im(z)
        limit(im_seq(neg_seq(z))) = -limit(im_seq(z))
    }
}

/// The real component limit of the conjugated sequence is the original real component limit.
theorem complex_limit_re_of_conj_seq(z: Nat -> Complex) {
    complex_converges(z) implies limit(re_seq(conj_seq(z))) = limit(re_seq(z))
} by {
    if complex_converges(z) {
        conj_seq_converges(z)
        complex_limit_re(conj_seq(z))
        complex_limit_of_conj_seq(z)
        re_conj(complex_limit(z))
        complex_limit_re(z)
        limit(re_seq(conj_seq(z))) = limit(re_seq(z))
    }
}

/// The imaginary component limit of the conjugated sequence is the negated original imaginary component limit.
theorem complex_limit_im_of_conj_seq(z: Nat -> Complex) {
    complex_converges(z) implies limit(im_seq(conj_seq(z))) = -limit(im_seq(z))
} by {
    if complex_converges(z) {
        conj_seq_converges(z)
        complex_limit_im(conj_seq(z))
        complex_limit_of_conj_seq(z)
        im_conj(complex_limit(z))
        complex_limit_im(z)
        limit(im_seq(conj_seq(z))) = -limit(im_seq(z))
    }
}

/// The real component limit of a pointwise sum is the sum of real component limits.
theorem complex_limit_re_of_add_seq(a: Nat -> Complex, b: Nat -> Complex) {
    complex_converges(a) and complex_converges(b)
    implies limit(re_seq(complex_add_seq(a, b))) = limit(re_seq(a)) + limit(re_seq(b))
} by {
    if complex_converges(a) and complex_converges(b) {
        complex_add_seq_converges(a, b)
        complex_limit_re(complex_add_seq(a, b))
        complex_limit_of_add_seq(a, b)
        re_add(complex_limit(a), complex_limit(b))
        complex_limit_re(a)
        complex_limit_re(b)
        limit(re_seq(complex_add_seq(a, b))) = complex_limit(complex_add_seq(a, b)).re
        limit(re_seq(complex_add_seq(a, b))) = limit(re_seq(a)) + limit(re_seq(b))
    }
}

/// The imaginary component limit of a pointwise sum is the sum of imaginary component limits.
theorem complex_limit_im_of_add_seq(a: Nat -> Complex, b: Nat -> Complex) {
    complex_converges(a) and complex_converges(b)
    implies limit(im_seq(complex_add_seq(a, b))) = limit(im_seq(a)) + limit(im_seq(b))
} by {
    if complex_converges(a) and complex_converges(b) {
        complex_add_seq_converges(a, b)
        complex_limit_im(complex_add_seq(a, b))
        complex_limit_of_add_seq(a, b)
        im_add(complex_limit(a), complex_limit(b))
        complex_limit_im(a)
        complex_limit_im(b)
        limit(im_seq(complex_add_seq(a, b))) = complex_limit(complex_add_seq(a, b)).im
        limit(im_seq(complex_add_seq(a, b))) = limit(im_seq(a)) + limit(im_seq(b))
    }
}

/// The real component limit of a pointwise difference is the difference of real component limits.
theorem complex_limit_re_of_sub_seq(a: Nat -> Complex, b: Nat -> Complex) {
    complex_converges(a) and complex_converges(b)
    implies limit(re_seq(complex_sub_seq(a, b))) = limit(re_seq(a)) - limit(re_seq(b))
} by {
    if complex_converges(a) and complex_converges(b) {
        complex_sub_seq_converges(a, b)
        complex_limit_re(complex_sub_seq(a, b))
        complex_limit_of_sub_seq(a, b)
        re_sub(complex_limit(a), complex_limit(b))
        complex_limit_re(a)
        complex_limit_re(b)
        limit(re_seq(complex_sub_seq(a, b))) = complex_limit(complex_sub_seq(a, b)).re
        limit(re_seq(complex_sub_seq(a, b))) = limit(re_seq(a)) - limit(re_seq(b))
    }
}

/// The imaginary component limit of a pointwise difference is the difference of imaginary component limits.
theorem complex_limit_im_of_sub_seq(a: Nat -> Complex, b: Nat -> Complex) {
    complex_converges(a) and complex_converges(b)
    implies limit(im_seq(complex_sub_seq(a, b))) = limit(im_seq(a)) - limit(im_seq(b))
} by {
    if complex_converges(a) and complex_converges(b) {
        complex_sub_seq_converges(a, b)
        complex_limit_im(complex_sub_seq(a, b))
        complex_limit_of_sub_seq(a, b)
        im_sub(complex_limit(a), complex_limit(b))
        complex_limit_im(a)
        complex_limit_im(b)
        limit(im_seq(complex_sub_seq(a, b))) = complex_limit(complex_sub_seq(a, b)).im
        limit(im_seq(complex_sub_seq(a, b))) = limit(im_seq(a)) - limit(im_seq(b))
    }
}

/// The real component limit of a pointwise product follows the complex multiplication formula.
theorem complex_limit_re_of_mul_seq(a: Nat -> Complex, b: Nat -> Complex) {
    complex_converges(a) and complex_converges(b)
    implies limit(re_seq(complex_mul_seq(a, b))) =
        limit(re_seq(a)) * limit(re_seq(b)) - limit(im_seq(a)) * limit(im_seq(b))
} by {
    if complex_converges(a) and complex_converges(b) {
        complex_mul_seq_converges(a, b)
        complex_limit_re(complex_mul_seq(a, b))
        complex_limit_of_mul_seq(a, b)
        re_mul(complex_limit(a), complex_limit(b))
        complex_limit_re(a)
        complex_limit_re(b)
        complex_limit_im(a)
        complex_limit_im(b)
        limit(re_seq(complex_mul_seq(a, b))) = complex_limit(complex_mul_seq(a, b)).re
        limit(re_seq(complex_mul_seq(a, b))) =
            limit(re_seq(a)) * limit(re_seq(b)) - limit(im_seq(a)) * limit(im_seq(b))
    }
}

/// The imaginary component limit of a pointwise product follows the complex multiplication formula.
theorem complex_limit_im_of_mul_seq(a: Nat -> Complex, b: Nat -> Complex) {
    complex_converges(a) and complex_converges(b)
    implies limit(im_seq(complex_mul_seq(a, b))) =
        limit(re_seq(a)) * limit(im_seq(b)) + limit(im_seq(a)) * limit(re_seq(b))
} by {
    if complex_converges(a) and complex_converges(b) {
        complex_mul_seq_converges(a, b)
        complex_limit_im(complex_mul_seq(a, b))
        complex_limit_of_mul_seq(a, b)
        im_mul(complex_limit(a), complex_limit(b))
        complex_limit_re(a)
        complex_limit_re(b)
        complex_limit_im(a)
        complex_limit_im(b)
        limit(im_seq(complex_mul_seq(a, b))) = complex_limit(complex_mul_seq(a, b)).im
        limit(im_seq(complex_mul_seq(a, b))) =
            limit(re_seq(a)) * limit(im_seq(b)) + limit(im_seq(a)) * limit(re_seq(b))
    }
}
