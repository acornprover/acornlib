/// The fundamental theorem of algebra: every nonconstant polynomial with
/// complex coefficients has at least one complex root.
///
/// First proven rigorously by d'Alembert (1746) and Gauss (1799).  Over the
/// real numbers the statement is false (`x^2 + 1` has no real root), which
/// is why the complex numbers are the natural setting.
///
/// The statement is recorded with the library's univariate polynomials
/// `Polynomial[Complex]` (finitely supported coefficient functions) and
/// evaluation `polynomial_eval(p, z)`.

from complex.complex import Complex, eq_by_components, re_mul, im_mul, mul_inverse,
    mul_one_right, mul_one_left, div_one, mul_assoc, mul_comm, distrib, left_distrib,
    mul_from_real, real_mul_lifts, real_add_lifts, from_real_one, from_real_eq_zero,
    inverse_mul_self, mul_right_cancel, mul_zero_right, add_comm, add_assoc,
    add_zero_left, add_zero_right, add_neg_cancel, neg_re, neg_im
from complex.complex_abs import complex_modulus, modulus_squared, modulus_nonneg,
    re_le_modulus, modulus_neg, abs_squared_eq, nonneg_eq_of_squares_eq
from real import Real, two, two_nonzero, one_half_positive, one_half_plus_one_half,
    sqrt_mul_self, sqrt_value_nonneg, mul_nonneg, square_nonneg, lte_antisymm,
    lte_trans, lt_trans, mul_sub_distrib_right, mul_sub_distrib_left, sub_both_eq_sub_add,
    sub_either_order, non_neg_imp_zero_lte, zero_lte_imp_non_neg, add_lte_add,
    lte_add_right, lte_add_left, real_mul_comm,
    sub_cancels, pos_imp_eq_abs, neg_distrib, neg_neg, abs_gte_zero
from algebra.field.field import field_mul_nonzero, field_square_nonzero
from algebra.ring.ring import mul_sub_left, mul_sub_right, mul_neg_left, mul_neg_right, mul_neg_neg
from order import lt_imp_lte, lte_refl
from algebra.add_group import inverse_add, inverse_inverse, inverse_left
from polynomial import Polynomial, polynomial_eval, polynomial_constant, polynomial_monomial,
    polynomial_eval_add, polynomial_eval_constant, polynomial_eval_zero,
    polynomial_support_bounded_by, polynomial_eval_eq_eval_bound_of_support_bounded,
    polynomial_eval_bound_eq_coeff_eval, polynomial_eval_bound,
    polynomial_monomial_coeff_self, polynomial_monomial_coeff_of_ne,
    coeff_eval, coeff_tail, coeff_zero_from, coeff_zero_from_at, coeff_zero_at
from nat import Nat
from nat import lt_or_lte, lt_suc, lt_and_lte, lt_not_symm
from semiring import Semiring
from algebra.zero import Zero

/// The real number four, written as two plus two (the digit Real.4 is not
/// defined in the library).
let real_four: Real = two + two

/// The linear polynomial `b * X + c` with complex coefficients.
define linear_polynomial(b: Complex, c: Complex) -> Polynomial[Complex] {
    polynomial_constant(c) + polynomial_monomial(Nat.1, b)
}

/// The quadratic polynomial `a * X^2 + b * X + c` with complex coefficients.
define quadratic_polynomial(a: Complex, b: Complex, c: Complex) -> Polynomial[Complex] {
    polynomial_constant(c) + polynomial_monomial(Nat.1, b) + polynomial_monomial(Nat.2, a)
}

/// The quadratic-formula root candidate for `a * X^2 + b * X + c`, given a
/// complex square root `w` of the discriminant `b^2 - 4ac`.
define quadratic_root(a: Complex, b: Complex, c: Complex, w: Complex) -> Complex {
    (w - b) * (Complex.from_real(two) * a).inverse
}

/// The complex number two is nonzero.
theorem two_ne_zero {
    Complex.from_real(two) != Complex.0
} by {
    from_real_eq_zero(two)
    Complex.from_real(two) = Complex.0 iff two = Real.0
    two_nonzero
    two != Real.0
}

/// Twice a nonzero complex number is nonzero.
theorem two_mul_ne_zero(a: Complex) {
    a != Complex.0 implies Complex.from_real(two) * a != Complex.0
} by {
    if a != Complex.0 {
        two_ne_zero
        field_mul_nonzero(Complex.from_real(two), a)
        Complex.from_real(two) * a != Complex.0
    }
}

/// The square of twice a number is four times its square.
theorem two_mul_square(a: Complex) {
    (Complex.from_real(two) * a) * (Complex.from_real(two) * a) =
        Complex.from_real(real_four) * a * a
} by {
    real_mul_lifts(two, two)
    Complex.from_real(two * two) = Complex.from_real(two) * Complex.from_real(two)
    two * two = real_four
    Complex.from_real(two) * Complex.from_real(two) = Complex.from_real(real_four)
    mul_comm(Complex.from_real(two), a)
    Complex.from_real(two) * a = a * Complex.from_real(two)
    (Complex.from_real(two) * a) * (Complex.from_real(two) * a) =
        (a * Complex.from_real(two)) * (a * Complex.from_real(two))
    mul_assoc(a, Complex.from_real(two), a * Complex.from_real(two))
    (a * Complex.from_real(two)) * (a * Complex.from_real(two)) =
        a * (Complex.from_real(two) * (a * Complex.from_real(two)))
    mul_assoc(Complex.from_real(two), a, Complex.from_real(two))
    Complex.from_real(two) * (a * Complex.from_real(two)) =
        (Complex.from_real(two) * a) * Complex.from_real(two)
    mul_comm(Complex.from_real(two), a)
    (Complex.from_real(two) * a) * Complex.from_real(two) =
        (a * Complex.from_real(two)) * Complex.from_real(two)
    mul_assoc(a, Complex.from_real(two), Complex.from_real(two))
    (a * Complex.from_real(two)) * Complex.from_real(two) =
        a * (Complex.from_real(two) * Complex.from_real(two))
    a * (Complex.from_real(two) * Complex.from_real(two)) =
        a * Complex.from_real(real_four)
    mul_comm(a, Complex.from_real(real_four))
    a * Complex.from_real(real_four) = Complex.from_real(real_four) * a
    (Complex.from_real(two) * a) * (Complex.from_real(two) * a) =
        Complex.from_real(real_four) * a * a
}

/// The complex number two is one plus one.
theorem complex_two_eq_one_plus_one {
    Complex.from_real(two) = Complex.1 + Complex.1
} by {
    real_add_lifts(Real.1, Real.1)
    Complex.from_real(Real.1 + Real.1) = Complex.from_real(Real.1) + Complex.from_real(Real.1)
    two = Real.1 + Real.1
    Complex.from_real(two) = Complex.from_real(Real.1) + Complex.from_real(Real.1)
    from_real_one
    Complex.from_real(Real.1) = Complex.1
    Complex.from_real(two) = Complex.1 + Complex.1
}

/// Doubling a complex number is adding it to itself.
theorem two_mul_eq_add_self(x: Complex) {
    Complex.from_real(two) * x = x + x
} by {
    complex_two_eq_one_plus_one
    Complex.from_real(two) = Complex.1 + Complex.1
    left_distrib(Complex.1, Complex.1, x)
    (Complex.1 + Complex.1) * x = Complex.1 * x + Complex.1 * x
    mul_one_left(x)
    Complex.1 * x = x
    (Complex.1 + Complex.1) * x = x + x
    Complex.from_real(two) * x = x + x
}

/// a - (b - c) = a - b + c
theorem sub_sub_right(a: Complex, b: Complex, c: Complex) {
    a - (b - c) = a - b + c
} by {
    a - (b - c) = a + -(b - c)
    b - c = b + -c
    a + -(b - c) = a + -(b + -c)
    inverse_add(b, -c)
    -(b + -c) = -(-c) + -b
    inverse_inverse(c)
    -(-c) = c
    a + (-(-c) + -b) = a + (c + -b)
    add_comm(c, -b)
    c + -b = -b + c
    a + (c + -b) = a + (-b + c)
    a + (-b + c) = a + -b + c
    a - b + c = a + -b + c
    a - (b - c) = a - b + c
}

/// -(two*x) + x = -x
theorem neg_two_mul_add_self(x: Complex) {
    -(Complex.from_real(two) * x) + x = -x
} by {
    two_mul_eq_add_self(x)
    Complex.from_real(two) * x = x + x
    -(Complex.from_real(two) * x) = -(x + x)
    inverse_add(x, x)
    -(x + x) = -x + -x
    -(Complex.from_real(two) * x) = -x + -x
    -(Complex.from_real(two) * x) + x = -x + -x + x
    add_assoc(-x, -x, x)
    (-x + -x) + x = -x + (-x + x)
    inverse_left(x)
    -x + x = Complex.0
    -x + (-x + x) = -x + Complex.0
    add_zero_right(-x)
    -x + Complex.0 = -x
    -x + -x + x = -x
    -(Complex.from_real(two) * x) + x = -x
}

/// -x + -x = -(two * x)
theorem neg_add_neg_eq_neg_two_mul(x: Complex) {
    -x + -x = -(Complex.from_real(two) * x)
} by {
    two_mul_eq_add_self(x)
    Complex.from_real(two) * x = x + x
    -(Complex.from_real(two) * x) = -(x + x)
    inverse_add(x, x)
    -(x + x) = -x + -x
    -(Complex.from_real(two) * x) = -x + -x
}

/// (p - two*x) + x = p - x
theorem sub_two_mul_add_self(p: Complex, x: Complex) {
    (p - Complex.from_real(two) * x) + x = p - x
} by {
    (p - Complex.from_real(two) * x) + x = (p + -(Complex.from_real(two) * x)) + x
    add_assoc(p, -(Complex.from_real(two) * x), x)
    (p + -(Complex.from_real(two) * x)) + x = p + (-(Complex.from_real(two) * x) + x)
    neg_two_mul_add_self(x)
    -(Complex.from_real(two) * x) + x = -x
    p + (-(Complex.from_real(two) * x) + x) = p + -x
    p + -x = p - x
    (p - Complex.from_real(two) * x) + x = p - x
}

/// (a - b) - a = -b
theorem sub_sub_self(a: Complex, b: Complex) {
    (a - b) - a = -b
} by {
    (a - b) - a = (a + -b) + -a
    add_assoc(a, -b, -a)
    (a + -b) + -a = a + (-b + -a)
    add_comm(-b, -a)
    -b + -a = -a + -b
    a + (-b + -a) = a + (-a + -b)
    add_assoc(a, -a, -b)
    a + (-a + -b) = (a + -a) + -b
    add_neg_cancel(a)
    a + -a = Complex.0
    (a + -a) + -b = Complex.0 + -b
    add_zero_left(-b)
    Complex.0 + -b = -b
    (a - b) - a = -b
}

/// Cancel middle terms: a + (x - y) + z - x = a + z - y
theorem cancel_middle(a0: Complex, x: Complex, y: Complex, z: Complex) {
    a0 + (x - y) + z - x = a0 + z - y
} by {
    add_comm(a0 + (x - y), z)
    (a0 + (x - y)) + z = z + (a0 + (x - y))
    add_assoc(z, a0, x - y)
    z + (a0 + (x - y)) = (z + a0) + (x - y)
    add_comm(z, a0)
    z + a0 = a0 + z
    (z + a0) + (x - y) = (a0 + z) + (x - y)
    a0 + (x - y) + z - x = (a0 + z) + (x - y) + -x
    add_assoc(a0 + z, x - y, -x)
    ((a0 + z) + (x - y)) + -x = (a0 + z) + ((x - y) + -x)
    sub_sub_self(x, y)
    (x - y) - x = -y
    (x - y) + -x = -y
    (a0 + z) + ((x - y) + -x) = (a0 + z) + -y
    a0 + (x - y) + z - x = a0 + z - y
}

/// x * ((r * y) * z) = (r * x) * y * z
theorem scalar_mul_rearrange(r: Complex, x: Complex, y: Complex, z: Complex) {
    x * ((r * y) * z) = (r * x) * y * z
} by {
    mul_assoc(x, r * y, z)
    x * ((r * y) * z) = (x * (r * y)) * z
    mul_assoc(x, r, y)
    x * (r * y) = (x * r) * y
    mul_comm(x, r)
    x * r = r * x
    (x * r) * y = (r * x) * y
    (x * (r * y)) * z = ((r * x) * y) * z
    mul_assoc(r * x, y, z)
    ((r * x) * y) * z = (r * x) * y * z
    x * ((r * y) * z) = (r * x) * y * z
}

/// a - b - c = a - (b + c)
theorem sub_sub_eq_sub_add(a: Complex, b: Complex, c: Complex) {
    a - b - c = a - (b + c)
} by {
    a - b - c = (a + -b) + -c
    add_assoc(a, -b, -c)
    (a + -b) + -c = a + (-b + -c)
    inverse_add(b, c)
    -(b + c) = -c + -b
    add_comm(-c, -b)
    -c + -b = -b + -c
    a + (-b + -c) = a + (-(b + c))
    a + -(b + c) = a - (b + c)
    a - b - c = a - (b + c)
}

/// a + (b - c + d) = a + b - c + d
theorem add_sum_sub_reassoc(a: Complex, b: Complex, c: Complex, d: Complex) {
    a + (b - c + d) = a + b - c + d
} by {
    add_assoc(a, b + -c, d)
    a + ((b + -c) + d) = (a + (b + -c)) + d
    add_assoc(a, b, -c)
    (a + b) + -c = a + (b + -c)
    (a + (b + -c)) + d = ((a + b) + -c) + d
    ((a + b) + -c) + d = a + b - c + d
    a + (b - c + d) = a + b - c + d
}

/// The two-step tail of the square-of-difference expansion, in subtraction form.
theorem square_sub_tail(a: Complex, b: Complex) {
    a * a - a * b - a * b + b * b = a * a - Complex.from_real(two) * (a * b) + b * b
} by {
    sub_sub_eq_sub_add(a * a, a * b, a * b)
    a * a - a * b - a * b = a * a - (a * b + a * b)
    a * a - a * b - a * b + b * b = a * a - (a * b + a * b) + b * b
    two_mul_eq_add_self(a * b)
    Complex.from_real(two) * (a * b) = a * b + a * b
    a * a - (a * b + a * b) + b * b = a * a - Complex.from_real(two) * (a * b) + b * b
    a * a - a * b - a * b + b * b = a * a - Complex.from_real(two) * (a * b) + b * b
}

/// The square of a difference expands to the quadratic form.
theorem square_sub(a: Complex, b: Complex) {
    (a - b) * (a - b) = a * a - Complex.from_real(two) * b * a + b * b
} by {
    mul_sub_right(a, b, a - b)
    (a - b) * (a - b) = a * (a - b) - b * (a - b)
    mul_sub_left(a, a, b)
    a * (a - b) = a * a - a * b
    mul_sub_left(b, a, b)
    b * (a - b) = b * a - b * b
    (a - b) * (a - b) = (a * a - a * b) - (b * a - b * b)
    mul_comm(b, a)
    b * a = a * b
    (a - b) * (a - b) = (a * a - a * b) - (a * b - b * b)
    sub_sub_right(a * a - a * b, a * b, b * b)
    (a * a - a * b) - (a * b - b * b) = a * a - a * b - a * b + b * b
    square_sub_tail(a, b)
    a * a - a * b - a * b + b * b = a * a - Complex.from_real(two) * (a * b) + b * b
    mul_comm(a, b)
    a * b = b * a
    Complex.from_real(two) * (a * b) = Complex.from_real(two) * (b * a)
    a * a - Complex.from_real(two) * (a * b) + b * b =
        a * a - Complex.from_real(two) * (b * a) + b * b
    mul_assoc(Complex.from_real(two), b, a)
    Complex.from_real(two) * (b * a) = Complex.from_real(two) * b * a
    a * a - Complex.from_real(two) * (b * a) + b * b =
        a * a - Complex.from_real(two) * b * a + b * b
    (a - b) * (a - b) = a * a - Complex.from_real(two) * b * a + b * b
}

/// The distribution of `a` over the squared difference.
theorem distribute_square_sub(a: Complex, b: Complex, w: Complex) {
    a * ((w - b) * (w - b)) =
        a * w * w - Complex.from_real(two) * a * b * w + a * b * b
} by {
    square_sub(w, b)
    (w - b) * (w - b) = w * w - Complex.from_real(two) * b * w + b * b
    a * ((w - b) * (w - b)) =
        a * (w * w - Complex.from_real(two) * b * w + b * b)
    distrib(a, w * w - Complex.from_real(two) * b * w, b * b)
    a * ((w * w - Complex.from_real(two) * b * w) + b * b) =
        a * (w * w - Complex.from_real(two) * b * w) + a * (b * b)
    mul_sub_left(a, w * w, Complex.from_real(two) * b * w)
    a * (w * w - Complex.from_real(two) * b * w) =
        a * (w * w) - a * (Complex.from_real(two) * b * w)
    scalar_mul_rearrange(Complex.from_real(two), a, b, w)
    a * ((Complex.from_real(two) * b) * w) = (Complex.from_real(two) * a) * b * w
    a * (w * w) = a * w * w
    a * (b * b) = a * b * b
    a * (w * w - Complex.from_real(two) * b * w + b * b) =
        a * w * w - Complex.from_real(two) * a * b * w + a * b * b
    a * ((w - b) * (w - b)) =
        a * w * w - Complex.from_real(two) * a * b * w + a * b * b
}

/// Substitution of the first two summands.
theorem substitution_two(a: Complex, b: Complex, c: Complex, w: Complex) {
    c * (Complex.from_real(real_four) * a * a) +
    b * (w - b) * (Complex.from_real(two) * a) +
    a * ((w - b) * (w - b)) =
        Complex.from_real(real_four) * a * a * c +
        Complex.from_real(two) * a * b * (w - b) +
        a * ((w - b) * (w - b))
} by {
    c * (Complex.from_real(real_four) * a * a) =
        Complex.from_real(real_four) * a * a * c
    b * (w - b) * (Complex.from_real(two) * a) =
        Complex.from_real(two) * a * b * (w - b)
    c * (Complex.from_real(real_four) * a * a) +
        b * (w - b) * (Complex.from_real(two) * a) +
        a * ((w - b) * (w - b)) =
        Complex.from_real(real_four) * a * a * c +
        Complex.from_real(two) * a * b * (w - b) +
        a * ((w - b) * (w - b))
}

/// Substitution of the third summand.
theorem substitution_third(a: Complex, b: Complex, c: Complex, w: Complex) {
    Complex.from_real(real_four) * a * a * c +
    Complex.from_real(two) * a * b * (w - b) +
    a * ((w - b) * (w - b)) =
        Complex.from_real(real_four) * a * a * c +
        Complex.from_real(two) * a * b * (w - b) +
        a * w * w - Complex.from_real(two) * a * b * w + a * b * b
} by {
    distribute_square_sub(a, b, w)
    a * ((w - b) * (w - b)) =
        a * w * w - Complex.from_real(two) * a * b * w + a * b * b
    Complex.from_real(real_four) * a * a * c +
        Complex.from_real(two) * a * b * (w - b) +
        a * ((w - b) * (w - b)) =
        Complex.from_real(real_four) * a * a * c +
        Complex.from_real(two) * a * b * (w - b) +
        (a * w * w - Complex.from_real(two) * a * b * w + a * b * b)
    add_sum_sub_reassoc(Complex.from_real(real_four) * a * a * c +
        Complex.from_real(two) * a * b * (w - b),
        a * w * w, Complex.from_real(two) * a * b * w, a * b * b)
    Complex.from_real(real_four) * a * a * c +
        Complex.from_real(two) * a * b * (w - b) +
        (a * w * w - Complex.from_real(two) * a * b * w + a * b * b) =
        Complex.from_real(real_four) * a * a * c +
        Complex.from_real(two) * a * b * (w - b) +
        a * w * w - Complex.from_real(two) * a * b * w + a * b * b
    Complex.from_real(real_four) * a * a * c +
        Complex.from_real(two) * a * b * (w - b) +
        a * ((w - b) * (w - b)) =
        Complex.from_real(real_four) * a * a * c +
        Complex.from_real(two) * a * b * (w - b) +
        a * w * w - Complex.from_real(two) * a * b * w + a * b * b
}

/// The fully substituted and expanded quadratic sum.
theorem expansion_connect(a: Complex, b: Complex, c: Complex, w: Complex) {
    c * (Complex.from_real(real_four) * a * a) +
    b * (w - b) * (Complex.from_real(two) * a) +
    a * ((w - b) * (w - b)) =
        Complex.from_real(real_four) * a * a * c +
        Complex.from_real(two) * a * b * (w - b) +
        a * w * w - Complex.from_real(two) * a * b * w + a * b * b
} by {
    substitution_two(a, b, c, w)
    c * (Complex.from_real(real_four) * a * a) +
        b * (w - b) * (Complex.from_real(two) * a) +
        a * ((w - b) * (w - b)) =
        Complex.from_real(real_four) * a * a * c +
        Complex.from_real(two) * a * b * (w - b) +
        a * ((w - b) * (w - b))
    substitution_third(a, b, c, w)
    Complex.from_real(real_four) * a * a * c +
        Complex.from_real(two) * a * b * (w - b) +
        a * ((w - b) * (w - b)) =
        Complex.from_real(real_four) * a * a * c +
        Complex.from_real(two) * a * b * (w - b) +
        a * w * w - Complex.from_real(two) * a * b * w + a * b * b
    c * (Complex.from_real(real_four) * a * a) +
        b * (w - b) * (Complex.from_real(two) * a) +
        a * ((w - b) * (w - b)) =
        Complex.from_real(real_four) * a * a * c +
        Complex.from_real(two) * a * b * (w - b) +
        a * w * w - Complex.from_real(two) * a * b * w + a * b * b
}

/// The expanded quadratic sum collapses to the sum of its leading terms.
theorem full_collapse(a: Complex, b: Complex, c: Complex, w: Complex) {
    Complex.from_real(real_four) * a * a * c +
    Complex.from_real(two) * a * b * (w - b) +
    a * w * w - Complex.from_real(two) * a * b * w + a * b * b =
        Complex.from_real(real_four) * a * a * c + a * w * w - a * b * b
} by {
    mul_sub_left(Complex.from_real(two) * a * b, w, b)
    (Complex.from_real(two) * a * b) * (w - b) =
        (Complex.from_real(two) * a * b) * w - (Complex.from_real(two) * a * b) * b
    (Complex.from_real(two) * a * b) * w = Complex.from_real(two) * a * b * w
    (Complex.from_real(two) * a * b) * b = Complex.from_real(two) * a * b * b
    cancel_middle(Complex.from_real(real_four) * a * a * c,
        Complex.from_real(two) * a * b * w,
        Complex.from_real(two) * a * b * b,
        a * w * w)
    Complex.from_real(real_four) * a * a * c +
        (Complex.from_real(two) * a * b * w - Complex.from_real(two) * a * b * b) +
        a * w * w - Complex.from_real(two) * a * b * w =
        Complex.from_real(real_four) * a * a * c + a * w * w -
        Complex.from_real(two) * a * b * b
    Complex.from_real(real_four) * a * a * c +
        Complex.from_real(two) * a * b * (w - b) +
        a * w * w - Complex.from_real(two) * a * b * w + a * b * b =
        Complex.from_real(real_four) * a * a * c + a * w * w -
        Complex.from_real(two) * a * b * b + a * b * b
    Complex.from_real(two) * a * b * b = Complex.from_real(two) * (a * b * b)
    Complex.from_real(real_four) * a * a * c + a * w * w -
        Complex.from_real(two) * a * b * b + a * b * b =
        Complex.from_real(real_four) * a * a * c + a * w * w -
        Complex.from_real(two) * (a * b * b) + a * b * b
    sub_two_mul_add_self(Complex.from_real(real_four) * a * a * c + a * w * w, a * b * b)
    Complex.from_real(real_four) * a * a * c + a * w * w -
        Complex.from_real(two) * (a * b * b) + a * b * b =
        Complex.from_real(real_four) * a * a * c + a * w * w - a * b * b
    Complex.from_real(real_four) * a * a * c +
        Complex.from_real(two) * a * b * (w - b) +
        a * w * w - Complex.from_real(two) * a * b * w + a * b * b =
        Complex.from_real(real_four) * a * a * c + a * w * w - a * b * b
}

/// The collapsed quadratic sum factors out the leading coefficient.
theorem factored_quadratic_sum(a: Complex, b: Complex, c: Complex, w: Complex) {
    Complex.from_real(real_four) * a * a * c + a * w * w - a * b * b =
        a * (w * w - b * b + Complex.from_real(real_four) * a * c)
} by {
    distrib(a, w * w - b * b, Complex.from_real(real_four) * a * c)
    a * (w * w - b * b + Complex.from_real(real_four) * a * c) =
        a * (w * w - b * b) + a * (Complex.from_real(real_four) * a * c)
    mul_sub_left(a, w * w, b * b)
    a * (w * w - b * b) = a * (w * w) - a * (b * b)
    a * (w * w - b * b) + a * (Complex.from_real(real_four) * a * c) =
        a * (w * w) - a * (b * b) + a * (Complex.from_real(real_four) * a * c)
    mul_assoc(a, w, w)
    a * (w * w) = a * w * w
    mul_assoc(a, b, b)
    a * (b * b) = a * b * b
    a * (w * w) - a * (b * b) + a * (Complex.from_real(real_four) * a * c) =
        a * w * w + -(a * b * b) + a * (Complex.from_real(real_four) * a * c)
    add_assoc(a * w * w, -(a * b * b), a * (Complex.from_real(real_four) * a * c))
    a * w * w + (-(a * b * b) + a * (Complex.from_real(real_four) * a * c)) =
        (a * w * w + a * (Complex.from_real(real_four) * a * c)) + -(a * b * b)
    mul_comm(a, Complex.from_real(real_four) * a)
    a * (Complex.from_real(real_four) * a) = (Complex.from_real(real_four) * a) * a
    mul_assoc(a, Complex.from_real(real_four) * a, c)
    (a * (Complex.from_real(real_four) * a)) * c = a * ((Complex.from_real(real_four) * a) * c)
    a * (Complex.from_real(real_four) * a * c) = Complex.from_real(real_four) * a * a * c
    a * w * w + a * (Complex.from_real(real_four) * a * c) =
        Complex.from_real(real_four) * a * a * c + a * w * w
    (a * w * w + a * (Complex.from_real(real_four) * a * c)) + -(a * b * b) =
        (Complex.from_real(real_four) * a * a * c + a * w * w) + -(a * b * b)
    Complex.from_real(real_four) * a * a * c + a * w * w - a * b * b =
        (Complex.from_real(real_four) * a * a * c + a * w * w) + -(a * b * b)
    a * (w * w - b * b + Complex.from_real(real_four) * a * c) =
        Complex.from_real(real_four) * a * a * c + a * w * w - a * b * b
}

/// The discriminant substitution makes the raw quadratic sum vanish.
theorem discriminant_zero(a: Complex, b: Complex, c: Complex, w: Complex) {
    w * w = b * b - Complex.from_real(real_four) * a * c implies
    w * w - b * b + Complex.from_real(real_four) * a * c = Complex.0
} by {
    if w * w = b * b - Complex.from_real(real_four) * a * c {
        sub_sub_self(b, Complex.from_real(real_four) * a * c)
        (b * b - Complex.from_real(real_four) * a * c) - b * b =
            -(Complex.from_real(real_four) * a * c)
        w * w - b * b + Complex.from_real(real_four) * a * c =
            (b * b - Complex.from_real(real_four) * a * c) - b * b +
            Complex.from_real(real_four) * a * c
        (b * b - Complex.from_real(real_four) * a * c) - b * b +
            Complex.from_real(real_four) * a * c =
            -(Complex.from_real(real_four) * a * c) + Complex.from_real(real_four) * a * c
        inverse_left(Complex.from_real(real_four) * a * c)
        -(Complex.from_real(real_four) * a * c) + Complex.from_real(real_four) * a * c = Complex.0
        w * w - b * b + Complex.from_real(real_four) * a * c = Complex.0
    }
}

/// The discriminant identity makes the factored quadratic vanish.
theorem quadratic_discriminant_zero(a: Complex, b: Complex, c: Complex, w: Complex) {
    w * w = b * b - Complex.from_real(real_four) * a * c implies
    a * (w * w - b * b + Complex.from_real(real_four) * a * c) = Complex.0
} by {
    if w * w = b * b - Complex.from_real(real_four) * a * c {
        discriminant_zero(a, b, c, w)
        w * w - b * b + Complex.from_real(real_four) * a * c = Complex.0
        a * (w * w - b * b + Complex.from_real(real_four) * a * c) = a * Complex.0
        mul_zero_right(a)
        a * Complex.0 = Complex.0
        a * (w * w - b * b + Complex.from_real(real_four) * a * c) = Complex.0
    }
}

/// The quadratic-formula root is a root of the quadratic.
theorem quadratic_root_identity(a: Complex, b: Complex, c: Complex, w: Complex) {
    a != Complex.0 and
    w * w = b * b - Complex.from_real(real_four) * a * c
    implies
    c + b * quadratic_root(a, b, c, w) +
        a * (quadratic_root(a, b, c, w) * quadratic_root(a, b, c, w)) = Complex.0
} by {
    if a != Complex.0 and w * w = b * b - Complex.from_real(real_four) * a * c {
        let d: Complex = Complex.from_real(two) * a
        let z: Complex = quadratic_root(a, b, c, w)
        two_mul_ne_zero(a)
        d != Complex.0
        inverse_mul_self(d)
        d.inverse * d = Complex.1
        mul_assoc(w - b, d.inverse, d)
        (w - b) * d.inverse * d = (w - b) * (d.inverse * d)
        (w - b) * (d.inverse * d) = (w - b) * Complex.1
        mul_one_right(w - b)
        (w - b) * Complex.1 = w - b
        z * d = w - b
        mul_assoc(z, d, d)
        z * (d * d) = (z * d) * d
        z * (d * d) = (w - b) * d
        z * z * (d * d) = (z * d) * (z * d)
        z * z * (d * d) = (w - b) * (w - b)
        distrib(c, b * z, a * (z * z))
        (c + b * z + a * (z * z)) * (d * d) =
            c * (d * d) + b * z * (d * d) + a * (z * z) * (d * d)
        (c + b * z + a * (z * z)) * (d * d) =
            c * (d * d) + b * (w - b) * d + a * (w - b) * (w - b)
        two_mul_square(a)
        d * d = Complex.from_real(real_four) * a * a
        c * (d * d) = c * (Complex.from_real(real_four) * a * a)
        b * (w - b) * d = b * (w - b) * (Complex.from_real(two) * a)
        mul_assoc(a, w - b, w - b)
        a * (w - b) * (w - b) = a * ((w - b) * (w - b))
        c * (d * d) + b * (w - b) * d + a * (w - b) * (w - b) =
            c * (Complex.from_real(real_four) * a * a) +
            b * (w - b) * (Complex.from_real(two) * a) +
            a * ((w - b) * (w - b))
        expansion_connect(a, b, c, w)
        c * (Complex.from_real(real_four) * a * a) +
            b * (w - b) * (Complex.from_real(two) * a) +
            a * ((w - b) * (w - b)) =
            Complex.from_real(real_four) * a * a * c +
            Complex.from_real(two) * a * b * (w - b) +
            a * w * w - Complex.from_real(two) * a * b * w + a * b * b
        full_collapse(a, b, c, w)
        Complex.from_real(real_four) * a * a * c +
            Complex.from_real(two) * a * b * (w - b) +
            a * w * w - Complex.from_real(two) * a * b * w + a * b * b =
            Complex.from_real(real_four) * a * a * c + a * w * w - a * b * b
        factored_quadratic_sum(a, b, c, w)
        Complex.from_real(real_four) * a * a * c + a * w * w - a * b * b =
            a * (w * w - b * b + Complex.from_real(real_four) * a * c)
        quadratic_discriminant_zero(a, b, c, w)
        a * (w * w - b * b + Complex.from_real(real_four) * a * c) = Complex.0
        (c + b * z + a * (z * z)) * (d * d) = Complex.0
        field_square_nonzero(d)
        d * d != Complex.0
        mul_right_cancel(c + b * z + a * (z * z), Complex.0, d * d)
        c + b * z + a * (z * z) = Complex.0
        z = quadratic_root(a, b, c, w)
        c + b * quadratic_root(a, b, c, w) +
            a * (quadratic_root(a, b, c, w) * quadratic_root(a, b, c, w)) = Complex.0
    }
}

/// A monomial is support-bounded by one more than its exponent.
theorem polynomial_monomial_support_bounded_by_suc[R: Semiring](n: Nat, r: R) {
    polynomial_support_bounded_by(Polynomial[R].monomial(n, r), n.suc)
} by {
    forall(k: Nat) {
        if k < n.suc {
            coeff_zero_from_at(Polynomial[R].monomial(n, r).coeff, n.suc, k) =
                (not k < n.suc implies coeff_zero_at(Polynomial[R].monomial(n, r).coeff, k))
            coeff_zero_from_at(Polynomial[R].monomial(n, r).coeff, n.suc, k)
        } else {
            not k < n.suc
            lt_or_lte(k, n.suc)
            n.suc <= k
            if k = n {
                lt_suc(n)
                n < n.suc
                lt_and_lte(n, n.suc, k)
                n < k
                lt_not_symm(n, k)
                not k < n
                lt_and_lte(k, n, k)
                k < k
                lt_not_symm(k, k)
                not k < k
                false
            }
            k != n
            polynomial_monomial_coeff_of_ne[R](n, r, k)
            Polynomial[R].monomial(n, r).coeff(k) = R.0
            coeff_zero_at(Polynomial[R].monomial(n, r).coeff, k)
            coeff_zero_from_at(Polynomial[R].monomial(n, r).coeff, n.suc, k) =
                (not k < n.suc implies coeff_zero_at(Polynomial[R].monomial(n, r).coeff, k))
            coeff_zero_from_at(Polynomial[R].monomial(n, r).coeff, n.suc, k)
        }
    }
    coeff_zero_from(Polynomial[R].monomial(n, r).coeff, n.suc) = forall(k: Nat) {
        coeff_zero_from_at(Polynomial[R].monomial(n, r).coeff, n.suc, k)
    }
    coeff_zero_from(Polynomial[R].monomial(n, r).coeff, n.suc)
    polynomial_support_bounded_by(Polynomial[R].monomial(n, r), n.suc) =
        coeff_zero_from(Polynomial[R].monomial(n, r).coeff, n.suc)
    polynomial_support_bounded_by(Polynomial[R].monomial(n, r), n.suc)
}

/// Bounded evaluation at bound zero is the zero of the coefficient ring.
theorem coeff_eval_nat_zero[R: Semiring](c: Nat -> R, x: R) {
    coeff_eval(c, x, Nat.0) = R.0
}

/// Bounded evaluation of a monomial of degree one at a point is scalar multiplication.
theorem polynomial_eval_bound_monomial_one(r: Complex, x: Complex) {
    polynomial_eval_bound(polynomial_monomial(Nat.1, r), x, Nat.0.suc.suc) = r * x
} by {
    polynomial_eval_bound_eq_coeff_eval(polynomial_monomial(Nat.1, r), x, Nat.0.suc.suc)
    coeff_eval(polynomial_monomial(Nat.1, r).coeff, x, Nat.0.suc.suc) =
        polynomial_monomial(Nat.1, r).coeff(Nat.0) +
        x * coeff_eval(coeff_tail(polynomial_monomial(Nat.1, r).coeff), x, Nat.0.suc)
    polynomial_monomial_coeff_of_ne[Complex](Nat.1, r, Nat.0)
    Nat.0 != Nat.1
    polynomial_monomial(Nat.1, r).coeff(Nat.0) = Complex.0
    coeff_eval(coeff_tail(polynomial_monomial(Nat.1, r).coeff), x, Nat.0.suc) =
        polynomial_monomial(Nat.1, r).coeff(Nat.1) +
        x * coeff_eval(coeff_tail(coeff_tail(polynomial_monomial(Nat.1, r).coeff)), x, Nat.0)
    polynomial_monomial(Nat.1, r).coeff(Nat.1) = r
    coeff_eval_nat_zero(coeff_tail(coeff_tail(polynomial_monomial(Nat.1, r).coeff)), x)
    coeff_eval(coeff_tail(coeff_tail(polynomial_monomial(Nat.1, r).coeff)), x, Nat.0) = Complex.0
    coeff_eval(polynomial_monomial(Nat.1, r).coeff, x, Nat.0.suc.suc) =
        Complex.0 + x * (r + x * Complex.0)
    Complex.0 + x * (r + x * Complex.0) = r * x
    polynomial_eval_bound(polynomial_monomial(Nat.1, r), x, Nat.0.suc.suc) = r * x
}

/// Evaluation of a monomial of degree one at a point is scalar multiplication.
theorem polynomial_eval_monomial_one(r: Complex, x: Complex) {
    polynomial_eval(polynomial_monomial(Nat.1, r), x) = r * x
} by {
    polynomial_monomial_support_bounded_by_suc[Complex](Nat.1, r)
    polynomial_support_bounded_by(polynomial_monomial(Nat.1, r), Nat.0.suc.suc)
    polynomial_eval_eq_eval_bound_of_support_bounded(
        polynomial_monomial(Nat.1, r), x, Nat.0.suc.suc)
    polynomial_eval_bound_monomial_one(r, x)
    polynomial_eval(polynomial_monomial(Nat.1, r), x) = r * x
}

/// Bounded evaluation of a monomial of degree two at a point is a scalar square.
theorem polynomial_eval_bound_monomial_two(r: Complex, x: Complex) {
    polynomial_eval_bound(polynomial_monomial(Nat.2, r), x, Nat.0.suc.suc.suc) = r * x * x
} by {
    polynomial_eval_bound_eq_coeff_eval(polynomial_monomial(Nat.2, r), x, Nat.0.suc.suc.suc)
    coeff_eval(polynomial_monomial(Nat.2, r).coeff, x, Nat.0.suc.suc.suc) =
        polynomial_monomial(Nat.2, r).coeff(Nat.0) +
        x * coeff_eval(coeff_tail(polynomial_monomial(Nat.2, r).coeff), x, Nat.0.suc.suc)
    polynomial_monomial_coeff_of_ne[Complex](Nat.2, r, Nat.0)
    Nat.0 != Nat.2
    polynomial_monomial(Nat.2, r).coeff(Nat.0) = Complex.0
    coeff_eval(coeff_tail(polynomial_monomial(Nat.2, r).coeff), x, Nat.0.suc.suc) =
        polynomial_monomial(Nat.2, r).coeff(Nat.1) +
        x * coeff_eval(coeff_tail(coeff_tail(polynomial_monomial(Nat.2, r).coeff)), x, Nat.0.suc)
    polynomial_monomial_coeff_of_ne[Complex](Nat.2, r, Nat.1)
    Nat.1 != Nat.2
    polynomial_monomial(Nat.2, r).coeff(Nat.1) = Complex.0
    coeff_tail(coeff_tail(polynomial_monomial(Nat.2, r).coeff))(Nat.0) =
        polynomial_monomial(Nat.2, r).coeff(Nat.2)
    coeff_eval(coeff_tail(coeff_tail(polynomial_monomial(Nat.2, r).coeff)), x, Nat.0.suc) =
        polynomial_monomial(Nat.2, r).coeff(Nat.2) +
        x * coeff_eval(coeff_tail(coeff_tail(coeff_tail(polynomial_monomial(Nat.2, r).coeff))), x, Nat.0)
    polynomial_monomial(Nat.2, r).coeff(Nat.2) = r
    coeff_eval_nat_zero(coeff_tail(coeff_tail(coeff_tail(polynomial_monomial(Nat.2, r).coeff))), x)
    coeff_eval(coeff_tail(coeff_tail(coeff_tail(polynomial_monomial(Nat.2, r).coeff))), x, Nat.0) = Complex.0
    coeff_eval(polynomial_monomial(Nat.2, r).coeff, x, Nat.0.suc.suc.suc) =
        Complex.0 + x * (Complex.0 + x * (r + x * Complex.0))
    Complex.0 + x * (Complex.0 + x * (r + x * Complex.0)) = r * x * x
    polynomial_eval_bound(polynomial_monomial(Nat.2, r), x, Nat.0.suc.suc.suc) = r * x * x
}

/// Evaluation of a monomial of degree two at a point is a scalar square.
theorem polynomial_eval_monomial_two(r: Complex, x: Complex) {
    polynomial_eval(polynomial_monomial(Nat.2, r), x) = r * x * x
} by {
    polynomial_monomial_support_bounded_by_suc[Complex](Nat.2, r)
    polynomial_support_bounded_by(polynomial_monomial(Nat.2, r), Nat.0.suc.suc.suc)
    polynomial_eval_eq_eval_bound_of_support_bounded(
        polynomial_monomial(Nat.2, r), x, Nat.0.suc.suc.suc)
    polynomial_eval_bound_monomial_two(r, x)
    polynomial_eval(polynomial_monomial(Nat.2, r), x) = r * x * x
}

/// A linear polynomial with nonzero leading coefficient has a root.
theorem fta_linear(b: Complex, c: Complex) {
    b != Complex.0 implies
    exists(z: Complex) {
        polynomial_eval(linear_polynomial(b, c), z) = Complex.0
    }
} by {
    if b != Complex.0 {
        let z0: Complex = -c * b.inverse
        mul_inverse(b)
        b * b.inverse = Complex.1
        polynomial_eval(linear_polynomial(b, c), z0) =
            polynomial_eval(polynomial_constant(c) + polynomial_monomial(Nat.1, b), z0)
        polynomial_eval(polynomial_constant(c) + polynomial_monomial(Nat.1, b), z0) =
            polynomial_eval(polynomial_constant(c), z0) + polynomial_eval(polynomial_monomial(Nat.1, b), z0)
        polynomial_eval(polynomial_constant(c), z0) = c
        polynomial_eval_monomial_one(b, z0)
        polynomial_eval(polynomial_monomial(Nat.1, b), z0) = b * z0
        mul_assoc(b, -c, b.inverse)
        b * (-c * b.inverse) = (b * -c) * b.inverse
        mul_comm(b, -c)
        b * -c = -c * b
        (b * -c) * b.inverse = (-c * b) * b.inverse
        mul_assoc(-c, b, b.inverse)
        (-c * b) * b.inverse = -c * (b * b.inverse)
        b * (-c * b.inverse) = -c * (b * b.inverse)
        b * b.inverse = Complex.1
        mul_one_right(-c)
        -c * Complex.1 = -c
        b * (-c * b.inverse) = -c
        c + -c = Complex.0
        polynomial_eval(linear_polynomial(b, c), z0) = c + b * z0
        c + b * z0 = Complex.0
        polynomial_eval(linear_polynomial(b, c), z0) = Complex.0
        exists(z: Complex) {
            polynomial_eval(linear_polynomial(b, c), z) = Complex.0
        }
    }
}

/// A quadratic polynomial with a complex square root of its discriminant has
/// a root, namely the quadratic-formula root `(-b + w) / (2a)`.
theorem fta_quadratic_of_discriminant_sqrt(a: Complex, b: Complex, c: Complex, w: Complex) {
    a != Complex.0 and w * w = b * b - Complex.from_real(real_four) * a * c implies
    exists(z: Complex) {
        polynomial_eval(quadratic_polynomial(a, b, c), z) = Complex.0
    }
} by {
    if a != Complex.0 and w * w = b * b - Complex.from_real(real_four) * a * c {
        let z: Complex = quadratic_root(a, b, c, w)
        quadratic_root_identity(a, b, c, w)
        c + b * quadratic_root(a, b, c, w) +
            a * (quadratic_root(a, b, c, w) * quadratic_root(a, b, c, w)) = Complex.0
        polynomial_eval(quadratic_polynomial(a, b, c), z) =
            polynomial_eval(polynomial_constant(c) + polynomial_monomial(Nat.1, b) +
                polynomial_monomial(Nat.2, a), z)
        polynomial_eval_add(polynomial_constant(c) + polynomial_monomial(Nat.1, b),
            polynomial_monomial(Nat.2, a), z)
        polynomial_eval(polynomial_constant(c) + polynomial_monomial(Nat.1, b) +
            polynomial_monomial(Nat.2, a), z) =
            polynomial_eval(polynomial_constant(c) + polynomial_monomial(Nat.1, b), z) +
            polynomial_eval(polynomial_monomial(Nat.2, a), z)
        polynomial_eval_add(polynomial_constant(c), polynomial_monomial(Nat.1, b), z)
        polynomial_eval(polynomial_constant(c) + polynomial_monomial(Nat.1, b), z) =
            polynomial_eval(polynomial_constant(c), z) +
            polynomial_eval(polynomial_monomial(Nat.1, b), z)
        polynomial_eval_constant(c, z)
        polynomial_eval(polynomial_constant(c), z) = c
        polynomial_eval_monomial_one(b, z)
        polynomial_eval(polynomial_monomial(Nat.1, b), z) = b * z
        polynomial_eval_monomial_two(a, z)
        polynomial_eval(polynomial_monomial(Nat.2, a), z) = a * z * z
        polynomial_eval(quadratic_polynomial(a, b, c), z) = c + b * z + a * z * z
        mul_assoc(a, z, z)
        a * z * z = a * (z * z)
        c + b * z + a * z * z = c + b * z + a * (z * z)
        z = quadratic_root(a, b, c, w)
        c + b * z + a * (z * z) =
            c + b * quadratic_root(a, b, c, w) +
            a * (quadratic_root(a, b, c, w) * quadratic_root(a, b, c, w))
        polynomial_eval(quadratic_polynomial(a, b, c), z) = Complex.0
        exists(r: Complex) {
            polynomial_eval(quadratic_polynomial(a, b, c), r) = Complex.0
        }
    }
}

/// Real multiplication is associative.
theorem real_mul_assoc(a: Real, b: Real, c: Real) {
    (a * b) * c = a * (b * c)
} by {}

/// Real multiplication distributes over addition on the left.
theorem real_left_distrib(a: Real, b: Real, c: Real) {
    (a + b) * c = a * c + b * c
} by {}

/// Real multiplication distributes over addition on the right.
theorem real_distrib(a: Real, b: Real, c: Real) {
    a * (b + c) = a * b + a * c
} by {}

/// The real one-half times two is one.
theorem one_half_mul_two {
    Real.one_half * two = Real.1
} by {
    two = Real.1 + Real.1
    Real.one_half * two = Real.one_half * (Real.1 + Real.1)
    Real.one_half * (Real.1 + Real.1) = Real.one_half * Real.1 + Real.one_half * Real.1
    Real.one_half * Real.1 = Real.one_half
    Real.one_half * Real.1 + Real.one_half * Real.1 = Real.one_half + Real.one_half
    one_half_plus_one_half
    Real.one_half + Real.one_half = Real.1
    Real.one_half * two = Real.1
}

/// A real number plus its negation is zero.
theorem real_add_neg_cancel(a: Real) {
    a + -a = Real.0
} by {
    sub_cancels(Real.0, a)
    Real.0 + a - a = Real.0
    Real.0 + a = a
    a - a = Real.0
    a - a = a + -a
    a + -a = Real.0
}

/// The product of a sum and a difference expands to a difference of squares.
theorem square_diff_real(a: Real, b: Real) {
    (a + b) * (a - b) = a * a - b * b
} by {
    mul_sub_distrib_right(a + b, a, b)
    (a + b) * (a - b) = (a + b) * a - (a + b) * b
    (a + b) * a = a * a + b * a
    (a + b) * b = a * b + b * b
    (a + b) * (a - b) = a * a + b * a - (a * b + b * b)
    real_mul_comm(b, a)
    b * a = a * b
    a * a + b * a - (a * b + b * b) = a * a + a * b - (a * b + b * b)
    sub_both_eq_sub_add(a * a + a * b, a * b, b * b)
    (a * a + a * b) - (a * b + b * b) = a * a + a * b - a * b - b * b
    sub_cancels(a * a, a * b)
    a * a + a * b - a * b = a * a
    a * a + a * b - a * b - b * b = a * a - b * b
    (a + b) * (a - b) = a * a - b * b
}

/// Four times the product of two halves is the product of the inputs.
theorem four_half_mul(a: Real, b: Real) {
    real_four * (Real.one_half * a) * (Real.one_half * b) = a * b
} by {
    real_four = two + two
    real_four * Real.one_half = (two + two) * Real.one_half
    (two + two) * Real.one_half = two * Real.one_half + two * Real.one_half
    one_half_mul_two
    Real.one_half * two = Real.1
    real_mul_comm(two, Real.one_half)
    two * Real.one_half = Real.one_half * two
    two * Real.one_half = Real.1
    two * Real.one_half + two * Real.one_half = Real.1 + Real.1
    real_four * Real.one_half = two
    real_four * (Real.one_half * a) = (real_four * Real.one_half) * a
    (real_four * Real.one_half) * a = two * a
    real_four * (Real.one_half * a) * (Real.one_half * b) = (two * a) * (Real.one_half * b)
    real_mul_assoc(two, a, Real.one_half * b)
    (two * a) * (Real.one_half * b) = two * (a * (Real.one_half * b))
    a * (Real.one_half * b) = Real.one_half * (a * b)
    two * (Real.one_half * (a * b)) = (two * Real.one_half) * (a * b)
    (two * Real.one_half) * (a * b) = Real.1 * (a * b)
    Real.1 * (a * b) = a * b
    real_four * (Real.one_half * a) * (Real.one_half * b) = a * b
}

/// The difference of the two halves is the second input.
theorem half_diff(a: Real, b: Real) {
    Real.one_half * (a + b) - Real.one_half * (a - b) = b
} by {
    mul_sub_distrib_right(Real.one_half, a + b, a - b)
    Real.one_half * ((a + b) - (a - b)) = Real.one_half * (a + b) - Real.one_half * (a - b)
    sub_both_eq_sub_add(a + b, a, -b)
    (a + b) - a - -b = (a + b) - (a + -b)
    (a + b) - (a + -b) = (a + b) - (a - b)
    neg_neg(b)
    -(-b) = b
    (a + b) - a - -b = (a + b) - a + b
    a + b = b + a
    sub_cancels(b, a)
    b + a - a = b
    (a + b) - a + b = b + b
    Real.one_half * ((a + b) - (a - b)) = Real.one_half * (b + b)
    Real.one_half * (b + b) = Real.one_half * (two * b)
    real_mul_assoc(Real.one_half, two, b)
    Real.one_half * (two * b) = (Real.one_half * two) * b
    one_half_mul_two
    Real.one_half * two = Real.1
    (Real.one_half * two) * b = Real.1 * b
    Real.1 * b = b
    Real.one_half * (a + b) - Real.one_half * (a - b) = b
}

/// Doubling a real number is adding it to itself.
theorem two_mul_eq_add_self_real(x: Real) {
    two * x = x + x
} by {
    two = Real.1 + Real.1
    real_left_distrib(Real.1, Real.1, x)
    (Real.1 + Real.1) * x = Real.1 * x + Real.1 * x
    Real.1 * x = x
    (Real.1 + Real.1) * x = x + x
    two * x = x + x
}

/// Twice a nonnegative real is nonnegative.
theorem two_mul_nonneg(a: Real) {
    a >= Real.0 implies two * a >= Real.0
} by {
    if a >= Real.0 {
        add_lte_add(Real.0, a, Real.0, a)
        Real.0 + Real.0 <= a + a
        Real.0 + Real.0 = Real.0
        Real.0 <= a + a
        two_mul_eq_add_self_real(a)
        two * a = a + a
        Real.0 <= two * a
        two * a >= Real.0
    }
}

/// The square of the doubled cross term is the square of the imaginary part.
theorem sqrt_components_square(u: Real, v: Real, m: Real, x: Real, y: Real) {
    u * u = Real.one_half * (m + x) and v * v = Real.one_half * (m - x) and
    m * m = x * x + y * y implies
    (two * (u * v)) * (two * (u * v)) = y * y
} by {
    if u * u = Real.one_half * (m + x) and v * v = Real.one_half * (m - x) and
        m * m = x * x + y * y {
        let t: Real = two * (u * v)
        t * t = (two * (u * v)) * (two * (u * v))
        real_mul_assoc(two, u, v)
        (two * u) * v = two * (u * v)
        (two * (u * v)) * (two * (u * v)) = ((two * u) * v) * ((two * u) * v)
        real_mul_comm(two, u)
        two * u = u * two
        ((two * u) * v) * ((two * u) * v) = ((u * two) * v) * ((u * two) * v)
        real_mul_assoc(u, two, v)
        (u * two) * v = u * (two * v)
        ((u * two) * v) * ((u * two) * v) = (u * (two * v)) * (u * (two * v))
        real_mul_assoc(u, two * v, u * (two * v))
        (u * (two * v)) * (u * (two * v)) = u * ((two * v) * (u * (two * v)))
        real_mul_assoc(two * v, u, two * v)
        (two * v) * (u * (two * v)) = ((two * v) * u) * (two * v)
        real_mul_comm(two * v, u)
        (two * v) * u = u * (two * v)
        ((two * v) * u) * (two * v) = (u * (two * v)) * (two * v)
        u * ((u * (two * v)) * (two * v)) = u * (u * ((two * v) * (two * v)))
        real_mul_assoc(u, u, (two * v) * (two * v))
        u * (u * ((two * v) * (two * v))) = (u * u) * ((two * v) * (two * v))
        (two * (u * v)) * (two * (u * v)) = (u * u) * ((two * v) * (two * v))
        real_mul_comm(two, v)
        two * v = v * two
        (two * v) * (two * v) = (v * two) * (v * two)
        real_mul_assoc(v, two, v * two)
        (v * two) * (v * two) = v * (two * (v * two))
        real_mul_assoc(two, v, two)
        two * (v * two) = (two * v) * two
        real_mul_comm(two, v)
        (two * v) * two = (v * two) * two
        real_mul_assoc(v, two, two)
        (v * two) * two = v * (two * two)
        v * (two * two) = v * real_four
        real_mul_comm(v, real_four)
        v * real_four = real_four * v
        (two * v) * (two * v) = real_four * v * v
        (u * u) * ((two * v) * (two * v)) = (u * u) * (real_four * (v * v))
        four_half_mul(m + x, m - x)
        real_four * (Real.one_half * (m + x)) * (Real.one_half * (m - x)) = (m + x) * (m - x)
        u * u = Real.one_half * (m + x)
        v * v = Real.one_half * (m - x)
        (u * u) * (real_four * (v * v)) = (Real.one_half * (m + x)) * (real_four * (Real.one_half * (m - x)))
        real_mul_comm(real_four, Real.one_half * (m - x))
        real_four * (Real.one_half * (m - x)) = (Real.one_half * (m - x)) * real_four
        (Real.one_half * (m + x)) * ((Real.one_half * (m - x)) * real_four) =
            (Real.one_half * (m + x)) * (Real.one_half * (m - x)) * real_four
        real_mul_assoc(Real.one_half * (m + x), Real.one_half * (m - x), real_four)
        ((Real.one_half * (m + x)) * (Real.one_half * (m - x))) * real_four =
            (Real.one_half * (m + x)) * ((Real.one_half * (m - x)) * real_four)
        real_mul_comm(Real.one_half * (m + x), Real.one_half * (m - x))
        (Real.one_half * (m + x)) * (Real.one_half * (m - x)) =
            (Real.one_half * (m - x)) * (Real.one_half * (m + x))
        ((Real.one_half * (m - x)) * (Real.one_half * (m + x))) * real_four =
            real_four * ((Real.one_half * (m - x)) * (Real.one_half * (m + x)))
        real_mul_comm((Real.one_half * (m - x)) * (Real.one_half * (m + x)), real_four)
        real_four * ((Real.one_half * (m - x)) * (Real.one_half * (m + x))) =
            real_four * (Real.one_half * (m - x)) * (Real.one_half * (m + x))
        real_mul_assoc(real_four, Real.one_half * (m - x), Real.one_half * (m + x))
        (real_four * (Real.one_half * (m - x))) * (Real.one_half * (m + x)) =
            real_four * ((Real.one_half * (m - x)) * (Real.one_half * (m + x)))
        real_four * (Real.one_half * (m - x)) * (Real.one_half * (m + x)) =
            real_four * (Real.one_half * (m + x)) * (Real.one_half * (m - x))
        real_mul_comm(Real.one_half * (m + x), Real.one_half * (m - x))
        (u * u) * (real_four * (v * v)) = (m + x) * (m - x)
        t * t = (m + x) * (m - x)
        square_diff_real(m, x)
        (m + x) * (m - x) = m * m - x * x
        t * t = m * m - x * x
        m * m = x * x + y * y
        t * t = x * x + y * y - x * x
        sub_cancels(y * y, x * x)
        y * y + x * x - x * x = y * y
        y * y + x * x = x * x + y * y
        x * x + y * y - x * x = y * y
        t * t = y * y
        (two * (u * v)) * (two * (u * v)) = y * y
    }
}

/// Every complex number has a complex square root.
///
/// Proof: for `z = (x, y)` with `m = |z|`, set `u = ((m + x) / 2).sqrt` and
/// `v = ((m - x) / 2).sqrt` using the real square root (both inputs are
/// nonnegative because `m >= x` and `m >= -x`).  Then `w = (u, v)` satisfies
/// `w^2 = (u^2 - v^2, 2uv) = (x, y) = z` when `y >= 0`, and `w = (u, -v)`
/// when `y < 0`.  The identity `2uv = |y|` follows from
/// `(2uv)^2 = (m + x)(m - x) = m^2 - x^2 = y^2` together with nonnegativity.
theorem complex_sqrt_square_exists(z: Complex) {
    exists(w: Complex) {
        w * w = z
    }
} by {
    let x: Real = z.re
    let y: Real = z.im
    let m: Real = z.modulus
    modulus_squared(z)
    m * m = z.abs_squared
    abs_squared_eq(z)
    z.abs_squared = x * x + y * y
    m * m = x * x + y * y
    modulus_nonneg(z)
    m >= Real.0
    re_le_modulus(z)
    x <= m
    re_le_modulus(-z)
    (-z).re <= (-z).modulus
    neg_re(z)
    (-z).re = -x
    modulus_neg(z)
    (-z).modulus = m
    -x <= m
    add_lte_add(-x, m, x, x)
    -x + x <= m + x
    real_add_neg_cancel(x)
    x + -x = Real.0
    -x + x = x + -x
    -x + x = Real.0
    Real.0 <= m + x
    m + x >= Real.0
    add_lte_add(x, m, -x, -x)
    x + -x <= m + -x
    Real.0 <= m + -x
    m + -x = m - x
    Real.0 <= m - x
    m - x >= Real.0
    one_half_positive
    Real.0 < Real.one_half
    lt_imp_lte(Real.0, Real.one_half)
    Real.0 <= Real.one_half
    Real.one_half >= Real.0
    mul_nonneg(Real.one_half, m + x)
    Real.one_half * (m + x) >= Real.0
    mul_nonneg(Real.one_half, m - x)
    Real.one_half * (m - x) >= Real.0
    sqrt_mul_self(Real.one_half * (m + x))
    exists(u: Real) {
        (Real.one_half * (m + x)).sqrt = Option.some(u) and u * u = Real.one_half * (m + x)
    }
    let u: Real satisfy {
        (Real.one_half * (m + x)).sqrt = Option.some(u) and u * u = Real.one_half * (m + x)
    }
    sqrt_mul_self(Real.one_half * (m - x))
    let v: Real satisfy {
        (Real.one_half * (m - x)).sqrt = Option.some(v) and v * v = Real.one_half * (m - x)
    }
    sqrt_value_nonneg(Real.one_half * (m + x), u)
    u >= Real.0
    sqrt_value_nonneg(Real.one_half * (m - x), v)
    v >= Real.0
    mul_nonneg(u, v)
    u * v >= Real.0
    sqrt_components_square(u, v, m, x, y)
    (two * (u * v)) * (two * (u * v)) = y * y
    two_mul_nonneg(u * v)
    two * (u * v) >= Real.0
    if y.is_negative {
        let w1: Complex = Complex.new(u, -v)
        re_mul(w1, w1)
        (w1 * w1).re = w1.re * w1.re - w1.im * w1.im
        w1.re = u
        w1.im = -v
        (w1 * w1).re = u * u - (-v) * (-v)
        u * u = Real.one_half * (m + x)
        v * v = Real.one_half * (m - x)
        mul_neg_neg(v, v)
        -v * -v = v * v
        (-v) * (-v) = v * v
        (w1 * w1).re = Real.one_half * (m + x) - Real.one_half * (m - x)
        half_diff(m, x)
        Real.one_half * (m + x) - Real.one_half * (m - x) = x
        (w1 * w1).re = x
        im_mul(w1, w1)
        (w1 * w1).im = w1.re * w1.im + w1.im * w1.re
        (w1 * w1).im = u * (-v) + (-v) * u
        mul_neg_right(u, v)
        u * -v = -(u * v)
        mul_neg_left(v, u)
        -v * u = -(v * u)
        real_mul_comm(v, u)
        v * u = u * v
        -v * u = -(u * v)
        (w1 * w1).im = -(u * v) + -(u * v)
        two_mul_eq_add_self_real(u * v)
        two * (u * v) = u * v + u * v
        neg_distrib(u * v, u * v)
        -(u * v + u * v) = -(u * v) + -(u * v)
        -(two * (u * v)) = -(u * v) + -(u * v)
        (w1 * w1).im = -(two * (u * v))
        y.is_negative
        (if y.is_negative { -y } else { y }) = y.abs
        (if y.is_negative { -y } else { y }) = -y
        y.abs = -y
        mul_neg_neg(y, y)
        -y * -y = y * y
        y.abs * y.abs = (-y) * (-y)
        y.abs * y.abs = y * y
        abs_gte_zero(y)
        y.abs >= Real.0
        nonneg_eq_of_squares_eq(two * (u * v), y.abs)
        two * (u * v) >= Real.0 and y.abs >= Real.0 and
            (two * (u * v)) * (two * (u * v)) = y.abs * y.abs
        two * (u * v) = y.abs
        two * (u * v) = -y
        neg_neg(y)
        -(-y) = y
        -(two * (u * v)) = -(-y)
        -(two * (u * v)) = y
        (w1 * w1).im = y
        z = Complex.new(x, y)
        eq_by_components(w1 * w1, z)
        w1 * w1 = z
        exists(w: Complex) {
            w * w = z
        }
    } else {
        not y.is_negative
        non_neg_imp_zero_lte(y)
        y >= Real.0
        let w2: Complex = Complex.new(u, v)
        re_mul(w2, w2)
        (w2 * w2).re = w2.re * w2.re - w2.im * w2.im
        w2.re = u
        w2.im = v
        (w2 * w2).re = u * u - v * v
        u * u = Real.one_half * (m + x)
        v * v = Real.one_half * (m - x)
        (w2 * w2).re = Real.one_half * (m + x) - Real.one_half * (m - x)
        half_diff(m, x)
        Real.one_half * (m + x) - Real.one_half * (m - x) = x
        (w2 * w2).re = x
        im_mul(w2, w2)
        (w2 * w2).im = w2.re * w2.im + w2.im * w2.re
        (w2 * w2).im = u * v + v * u
        real_mul_comm(v, u)
        v * u = u * v
        (w2 * w2).im = u * v + u * v
        two_mul_eq_add_self_real(u * v)
        two * (u * v) = u * v + u * v
        (w2 * w2).im = two * (u * v)
        (if y.is_negative { -y } else { y }) = y.abs
        (if y.is_negative { -y } else { y }) = y
        y.abs = y
        mul_neg_neg(y, y)
        -y * -y = y * y
        y.abs * y.abs = y * y
        abs_gte_zero(y)
        y.abs >= Real.0
        nonneg_eq_of_squares_eq(two * (u * v), y.abs)
        two * (u * v) >= Real.0 and y.abs >= Real.0 and
            (two * (u * v)) * (two * (u * v)) = y.abs * y.abs
        two * (u * v) = y.abs
        two * (u * v) = y
        (w2 * w2).im = y
        z = Complex.new(x, y)
        eq_by_components(w2 * w2, z)
        w2 * w2 = z
        exists(w: Complex) {
            w * w = z
        }
    }
}

/// Every quadratic polynomial with complex coefficients has a complex root:
/// the quadratic formula with a complex square root of the discriminant.
theorem fta_quadratic(a: Complex, b: Complex, c: Complex) {
    a != Complex.0 implies
    exists(z: Complex) {
        polynomial_eval(quadratic_polynomial(a, b, c), z) = Complex.0
    }
} by {
    if a != Complex.0 {
        complex_sqrt_square_exists(b * b - Complex.from_real(real_four) * a * c)
        exists(w: Complex) {
            w * w = b * b - Complex.from_real(real_four) * a * c
        }
        let w: Complex satisfy {
            w * w = b * b - Complex.from_real(real_four) * a * c
        }
        fta_quadratic_of_discriminant_sqrt(a, b, c, w)
        exists(z: Complex) {
            polynomial_eval(quadratic_polynomial(a, b, c), z) = Complex.0
        }
    }
}

/// The fundamental theorem of algebra: every nonconstant polynomial with
/// complex coefficients has at least one complex root.
///
/// A polynomial is nonconstant when it takes two distinct values, written
/// `exists(x, y) { polynomial_eval(p, x) != polynomial_eval(p, y) }`.
///
/// Proof sketch: three classical routes.  (1) Via Liouville's theorem: if
/// `p` had no root, `1 / p` would be a bounded entire function, hence
/// constant.  (2) Via the argument principle / Rouché's theorem: for a
/// polynomial `p(z) = a_n z^n + ... + a_0` with `a_n != 0`, on a large
/// circle the term `a_n z^n` dominates, so `p` and `a_n z^n` have the same
/// number of zeros inside.  (3) Via the fundamental group: a polynomial
/// map from the Riemann sphere to itself has degree `n`, so it is
/// surjective.  All three routes need complex analysis machinery
/// (holomorphic functions, contour integration, or the topology of the
/// sphere) that is not yet in the library.
///
/// The linear case (`fta_linear`) and the quadratic case (`fta_quadratic`,
/// via the quadratic formula and `complex_sqrt_square_exists`) are proved
/// above.  The general case is developed in `complex/fta_general.ac`
/// following the elementary minimum-modulus route:
///
///   - `polynomial_eval_growth`: `|p(z)|` grows to infinity (boundedness
///     on disks and the growth theorem);
///   - `fta_perturbation` (D'Alembert's lemma): if `p(z0) != 0` and `p`
///     is nonconstant, some nearby point evaluates to strictly smaller
///     modulus, via the iterated-quotient expansion of `p` around `z0`
///     and a perturbation direction chosen among the powers of
///     `omega(4k)`;
///   - `fundamental_theorem_of_algebra_of_extreme`: a global minimum of
///     `|p|` is a root.
///
/// The one remaining ingredient is the extreme value theorem for `|p|` on
/// a closed disk of the complex plane (two-dimensional compactness), which
/// the library does not yet provide; see Section 9 of
/// `complex/fta_general.ac` for the precise gap.
// theorem fundamental_theorem_of_algebra(p: Polynomial[Complex]) {
//     exists(x: Complex, y: Complex) {
//         polynomial_eval(p, x) != polynomial_eval(p, y)
//     } implies exists(z: Complex) {
//         polynomial_eval(p, z) = Complex.0
//     }
// }
