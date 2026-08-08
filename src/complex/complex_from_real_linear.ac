from real import Real
from algebra.module.module import Module, ring_as_module, ring_as_module_smul
from algebra.module.module_hom import is_linear_map, preserves_add, preserves_smul, ModuleHom
from complex.complex import Complex, real_add_lifts, complex_real_smul
from complex.complex_module import complex_real_module, complex_real_module_smul

/// `Complex.from_real` preserves addition.
theorem complex_from_real_preserves_add {
    preserves_add(Complex.from_real)
} by {
    forall(a: Real, b: Real) {
        real_add_lifts(a, b)
        Complex.from_real(a + b) = Complex.from_real(a) + Complex.from_real(b)
    }
}

/// Helper: per-input form of the real-scalar preservation equality for `Complex.from_real`.
theorem complex_from_real_smul_eq(r: Real, x: Real) {
    Complex.from_real(ring_as_module[Real].smul(r, x)) =
        complex_real_module.smul(r, Complex.from_real(x))
} by {
    ring_as_module_smul[Real](r, x)
    complex_real_module_smul(r, Complex.from_real(x))
    complex_real_smul(r, Complex.from_real(x)) =
        Complex.new(r * Complex.from_real(x).re, r * Complex.from_real(x).im)
    r * Real.0 = Real.0
}

/// `Complex.from_real` preserves the real scalar action.
theorem complex_from_real_preserves_smul {
    preserves_smul(ring_as_module[Real], complex_real_module, Complex.from_real)
} by {
    forall(r: Real, x: Real) {
        complex_from_real_smul_eq(r, x)
    }
}

/// `Complex.from_real` is a real-linear map from the reals to the complex numbers.
theorem complex_from_real_is_linear_map {
    is_linear_map(ring_as_module[Real], complex_real_module, Complex.from_real)
} by {
    complex_from_real_preserves_add
    complex_from_real_preserves_smul
}

/// `ModuleHom.new` produces a `Some` value for the real-to-complex embedding.
theorem complex_from_real_module_hom_some {
    exists(h: ModuleHom[Real, Real, Complex]) {
        ModuleHom.new(ring_as_module[Real], complex_real_module, Complex.from_real)
            = Option.some(h)
    }
} by {
    complex_from_real_is_linear_map
}

/// The real-to-complex embedding packaged as a real-linear `ModuleHom`.
let complex_from_real_module_hom: ModuleHom[Real, Real, Complex] satisfy {
    ModuleHom.new(ring_as_module[Real], complex_real_module, Complex.from_real) = Option.some(complex_from_real_module_hom)
}

/// The source module of `complex_from_real_module_hom` is `ring_as_module[Real]`.
theorem complex_from_real_module_hom_src {
    complex_from_real_module_hom.src = ring_as_module[Real]
} by {
    ModuleHom.new(ring_as_module[Real], complex_real_module, Complex.from_real) = Option.some(complex_from_real_module_hom)
}

/// The destination module of `complex_from_real_module_hom` is `complex_real_module`.
theorem complex_from_real_module_hom_dst {
    complex_from_real_module_hom.dst = complex_real_module
} by {
    ModuleHom.new(ring_as_module[Real], complex_real_module, Complex.from_real) = Option.some(complex_from_real_module_hom)
}

