from real import Real
from complex.complex import Complex, real_add_lifts, real_mul_lifts, from_real_one
from algebra.ring.ring_hom import RingHom, is_ring_hom, ring_hom_eq_of_hom_eq
from algebra.field.field_hom import FieldHom, is_field_hom, field_hom_to_ring_hom,
    field_hom_to_ring_hom_hom

/// The real-to-complex embedding preserves addition pointwise.
theorem complex_from_real_add(a: Real, b: Real) {
    Complex.from_real(a + b) = Complex.from_real(a) + Complex.from_real(b)
} by {
    real_add_lifts(a, b)
}

/// The real-to-complex embedding preserves multiplication pointwise.
theorem complex_from_real_mul(a: Real, b: Real) {
    Complex.from_real(a * b) = Complex.from_real(a) * Complex.from_real(b)
} by {
    real_mul_lifts(a, b)
}

/// The real-to-complex embedding sends one to one.
theorem complex_from_real_one {
    Complex.from_real(Real.1) = Complex.1
} by {
    from_real_one
}

/// The real-to-complex embedding preserves the ring operations and one.
theorem complex_from_real_is_ring_hom {
    is_ring_hom(Complex.from_real)
} by {
    complex_from_real_one
    forall(a: Real, b: Real) {
        complex_from_real_add(a, b)
        complex_from_real_mul(a, b)
        Complex.from_real(a + b) = Complex.from_real(a) + Complex.from_real(b) and
            Complex.from_real(a * b) = Complex.from_real(a) * Complex.from_real(b)
    }
}

/// `RingHom.new` produces a `Some` value for the real-to-complex embedding.
theorem complex_from_real_ring_hom_some {
    exists(h: RingHom[Real, Complex]) {
        RingHom.new(Complex.from_real) = Option.some(h)
    }
} by {
    complex_from_real_is_ring_hom
}

/// The real-to-complex embedding as a bundled ring homomorphism.
let complex_from_real_ring_hom: RingHom[Real, Complex] satisfy {
    RingHom.new(Complex.from_real) = Option.some(complex_from_real_ring_hom)
}

/// The underlying function of the bundled real-to-complex ring homomorphism.
theorem complex_from_real_ring_hom_hom {
    complex_from_real_ring_hom.hom = Complex.from_real
} by {
}

/// The real-to-complex embedding is a field homomorphism.
theorem complex_from_real_is_field_hom {
    is_field_hom(Complex.from_real)
} by {
    complex_from_real_is_ring_hom
}

/// `FieldHom.new` produces a `Some` value for the real-to-complex embedding.
theorem complex_from_real_field_hom_some {
    exists(h: FieldHom[Real, Complex]) {
        FieldHom.new(Complex.from_real) = Option.some(h)
    }
} by {
    complex_from_real_is_field_hom
}

/// The real-to-complex embedding as a bundled field homomorphism.
let complex_from_real_field_hom: FieldHom[Real, Complex] satisfy {
    FieldHom.new(Complex.from_real) = Option.some(complex_from_real_field_hom)
}

/// The underlying function of the bundled real-to-complex field homomorphism.
theorem complex_from_real_field_hom_hom {
    complex_from_real_field_hom.hom = Complex.from_real
} by {
}

/// The ring homomorphism induced by the field package agrees with the direct ring package.
theorem complex_from_real_field_hom_to_ring_hom {
    field_hom_to_ring_hom(complex_from_real_field_hom) = complex_from_real_ring_hom
} by {
    let lhs = field_hom_to_ring_hom(complex_from_real_field_hom)
    let rhs = complex_from_real_ring_hom
    field_hom_to_ring_hom_hom(complex_from_real_field_hom)
    complex_from_real_field_hom_hom
    complex_from_real_ring_hom_hom
    lhs.hom = rhs.hom
    ring_hom_eq_of_hom_eq(lhs, rhs)
}
