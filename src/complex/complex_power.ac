from real import Real
from complex.complex import Complex, mul_one_left, mul_one_right, mul_zero_right, from_real_zero, from_real_one, real_mul_lifts
from complex.complex_abs import modulus_of_one
from complex.complex_exp import complex_exp, complex_exp_from_real, complex_exp_zero
from complex.complex_log import complex_log, complex_log_from_real_pos, complex_exp_log_pos_cond, complex_log_exp_real_cond

// ---------------------------------------------------------------------------
// The complex power.
//
// For complex numbers z and w, the power z^w is defined as (w · z.log).exp,
// using the principal branch of the complex logarithm.  On the positive real
// axis this agrees with the real power: for x > 0, x^w = (w · ln(x)).exp, where
// x.log.get_or_else(Real.0) plays the role of the real logarithm ln(x).
//
// The real package's interface (src/real/interface.ac) exports the real
// exponential Real.exp, the real logarithm log_or_zero and the real power rpow, but
// no theorems about them (the proofs in src/real/Real.exp.ac and src/real/log.ac are
// private to the real package).  The results below that need only the complex
// definitions are proved unconditionally; those that additionally need a real
// fact ((x.log.get_or_else(Real.0)).exp = x, (Real.1).log.get_or_else(Real.0) = Real.0, (Real.1).exp > 0,
// ((Real.1).exp).log.get_or_else(Real.0) = Real.1) are stated with that fact as a hypothesis,
// exactly as in src/complex/complex_log.ac.
// ---------------------------------------------------------------------------

/// The complex power of z with exponent w: z^w = (w · z.log).exp, with the
/// principal branch of the complex logarithm.
define complex_pow(z: Complex, w: Complex) -> Complex {
    complex_exp(w * complex_log(z))
}

// ---------------------------------------------------------------------------
// Consistency with the real power on the positive real axis.
// ---------------------------------------------------------------------------

/// The complex power of an embedded positive real number x with an arbitrary
/// complex exponent w is (w · ln(x)).exp: the principal-branch value of x^w.
theorem complex_pow_from_real_pos(x: Real, w: Complex) {
    x > Real.0 implies complex_pow(Complex.from_real(x), w) =
        complex_exp(w * Complex.from_real(x.log.get_or_else(Real.0)))
} by {
    if x > Real.0 {
        complex_pow(Complex.from_real(x), w) = complex_exp(w * complex_log(Complex.from_real(x)))
        complex_log_from_real_pos(x)
        x > Real.0 implies complex_log(Complex.from_real(x)) = Complex.from_real(x.log.get_or_else(Real.0))
        complex_log(Complex.from_real(x)) = Complex.from_real(x.log.get_or_else(Real.0))
        w * complex_log(Complex.from_real(x)) = w * Complex.from_real(x.log.get_or_else(Real.0))
        complex_exp(w * complex_log(Complex.from_real(x))) =
            complex_exp(w * Complex.from_real(x.log.get_or_else(Real.0)))
        complex_pow(Complex.from_real(x), w) =
            complex_exp(w * Complex.from_real(x.log.get_or_else(Real.0)))
    }
}

/// The complex power of an embedded positive real number x with an embedded real
/// exponent y is the embedding of the real power: x^y = (y · ln(x)).exp.
theorem complex_pow_from_real(x: Real, y: Real) {
    x > Real.0 implies complex_pow(Complex.from_real(x), Complex.from_real(y)) =
        Complex.from_real((y * x.log.get_or_else(Real.0)).exp)
} by {
    if x > Real.0 {
        complex_pow_from_real_pos(x, Complex.from_real(y))
        x > Real.0 implies complex_pow(Complex.from_real(x), Complex.from_real(y)) =
            complex_exp(Complex.from_real(y) * Complex.from_real(x.log.get_or_else(Real.0)))
        complex_pow(Complex.from_real(x), Complex.from_real(y)) =
            complex_exp(Complex.from_real(y) * Complex.from_real(x.log.get_or_else(Real.0)))
        real_mul_lifts(y, x.log.get_or_else(Real.0))
        Complex.from_real(y * x.log.get_or_else(Real.0)) =
            Complex.from_real(y) * Complex.from_real(x.log.get_or_else(Real.0))
        Complex.from_real(y) * Complex.from_real(x.log.get_or_else(Real.0)) =
            Complex.from_real(y * x.log.get_or_else(Real.0))
        complex_exp(Complex.from_real(y) * Complex.from_real(x.log.get_or_else(Real.0))) =
            complex_exp(Complex.from_real(y * x.log.get_or_else(Real.0)))
        complex_exp_from_real(y * x.log.get_or_else(Real.0))
        complex_exp(Complex.from_real(y * x.log.get_or_else(Real.0))) =
            Complex.from_real((y * x.log.get_or_else(Real.0)).exp)
        complex_exp(Complex.from_real(y) * Complex.from_real(x.log.get_or_else(Real.0))) =
            Complex.from_real((y * x.log.get_or_else(Real.0)).exp)
        complex_pow(Complex.from_real(x), Complex.from_real(y)) =
            Complex.from_real((y * x.log.get_or_else(Real.0)).exp)
    }
}

/// The complex power of a positive real base with a real exponent agrees with
/// the real power function: whenever x.rpow(y) = z, the complex power is the
/// embedding of z.
theorem complex_pow_from_real_rpow(x: Real, y: Real, z: Real) {
    x > Real.0 and x.rpow(y) = Option.some(z)
    implies complex_pow(Complex.from_real(x), Complex.from_real(y)) = Complex.from_real(z)
} by {
    if x > Real.0 and x.rpow(y) = Option.some(z) {
        complex_pow_from_real(x, y)
        x > Real.0 implies complex_pow(Complex.from_real(x), Complex.from_real(y)) =
            Complex.from_real((y * x.log.get_or_else(Real.0)).exp)
        complex_pow(Complex.from_real(x), Complex.from_real(y)) =
            Complex.from_real((y * x.log.get_or_else(Real.0)).exp)
        x.rpow(y) = Option.some((y * x.log.get_or_else(Real.0)).exp)
        x.rpow(y) = Option.some(z)
        Option.some((y * x.log.get_or_else(Real.0)).exp) = Option.some(z)
        some_injective((y * x.log.get_or_else(Real.0)).exp, z)
        (y * x.log.get_or_else(Real.0)).exp = z
        Complex.from_real((y * x.log.get_or_else(Real.0)).exp) = Complex.from_real(z)
        complex_pow(Complex.from_real(x), Complex.from_real(y)) = Complex.from_real(z)
    }
}

// ---------------------------------------------------------------------------
// The inverse laws.
// ---------------------------------------------------------------------------

/// The complex exponential inverts the complex logarithm on the positive real
/// axis: (x.log).exp = x for every positive real x, whenever the real logarithm
/// is a right inverse of the real exponential on positive reals.
theorem complex_exp_log_pos_axis(x: Real) {
    x > Real.0 and (x.log.get_or_else(Real.0)).exp = x
    implies complex_exp(complex_log(Complex.from_real(x))) = Complex.from_real(x)
} by {
    complex_exp_log_pos_cond(x)
}

/// Raising a positive real number to the first power gives the number itself:
/// x^1 = x, whenever the real logarithm is a right inverse of the real
/// exponential on positive reals.
theorem complex_pow_pos_axis_one(x: Real) {
    x > Real.0 and (x.log.get_or_else(Real.0)).exp = x
    implies complex_pow(Complex.from_real(x), Complex.1) = Complex.from_real(x)
} by {
    if x > Real.0 and (x.log.get_or_else(Real.0)).exp = x {
        complex_pow(Complex.from_real(x), Complex.1) =
            complex_exp(Complex.1 * complex_log(Complex.from_real(x)))
        mul_one_left(complex_log(Complex.from_real(x)))
        Complex.1 * complex_log(Complex.from_real(x)) = complex_log(Complex.from_real(x))
        complex_exp(Complex.1 * complex_log(Complex.from_real(x))) =
            complex_exp(complex_log(Complex.from_real(x)))
        complex_exp_log_pos_cond(x)
        x > Real.0 and (x.log.get_or_else(Real.0)).exp = x implies complex_exp(complex_log(Complex.from_real(x))) = Complex.from_real(x)
        complex_exp(complex_log(Complex.from_real(x))) = Complex.from_real(x)
        complex_exp(Complex.1 * complex_log(Complex.from_real(x))) = Complex.from_real(x)
        complex_pow(Complex.from_real(x), Complex.1) = Complex.from_real(x)
    }
}

/// The complex logarithm inverts the complex exponential on the real axis:
/// (x.exp).log = x for every real x whose exponential value the real logarithm
/// recovers.  The restriction is necessary: the complex logarithm always yields
/// a real number, so (z.exp).log = z cannot hold off the real axis.
theorem complex_log_exp_real_axis(x: Real) {
    x.exp > Real.0 and (x.exp).log.get_or_else(Real.0) = x
    implies complex_log(complex_exp(Complex.from_real(x))) = Complex.from_real(x)
} by {
    complex_log_exp_real_cond(x)
}

// ---------------------------------------------------------------------------
// Special values.
// ---------------------------------------------------------------------------

/// The complex power of one is one: 1^w = 1, whenever the real logarithm of one
/// vanishes.
theorem complex_pow_one(w: Complex) {
    (Real.1).log.get_or_else(Real.0) = Real.0 implies complex_pow(Complex.1, w) = Complex.1
} by {
    if (Real.1).log.get_or_else(Real.0) = Real.0 {
        complex_pow(Complex.1, w) = complex_exp(w * complex_log(Complex.1))
        complex_log(Complex.1) = Complex.from_real((Complex.1.modulus).log.get_or_else(Real.0))
        modulus_of_one
        Complex.1.modulus = Real.1
        (Complex.1.modulus).log.get_or_else(Real.0) = (Real.1).log.get_or_else(Real.0)
        (Real.1).log.get_or_else(Real.0) = Real.0
        Complex.from_real((Complex.1.modulus).log.get_or_else(Real.0)) = Complex.from_real(Real.0)
        from_real_zero
        Complex.from_real(Real.0) = Complex.0
        Complex.from_real((Complex.1.modulus).log.get_or_else(Real.0)) = Complex.0
        complex_log(Complex.1) = Complex.0
        w * complex_log(Complex.1) = w * Complex.0
        mul_zero_right(w)
        w * Complex.0 = Complex.0
        complex_exp(w * complex_log(Complex.1)) = complex_exp(Complex.0)
        complex_exp_zero
        complex_exp(Complex.0) = Complex.1
        complex_exp(w * complex_log(Complex.1)) = Complex.1
        complex_pow(Complex.1, w) = Complex.1
    }
}

/// The complex power of e = (1).exp is the complex exponential: e^w = w.exp,
/// whenever the real logarithm recovers the exponential value at one.
theorem complex_pow_e(w: Complex) {
    (Real.1).exp > Real.0 and ((Real.1).exp).log.get_or_else(Real.0) = Real.1
    implies complex_pow(Complex.from_real((Real.1).exp), w) = complex_exp(w)
} by {
    if (Real.1).exp > Real.0 and ((Real.1).exp).log.get_or_else(Real.0) = Real.1 {
        complex_pow(Complex.from_real((Real.1).exp), w) =
            complex_exp(w * complex_log(Complex.from_real((Real.1).exp)))
        complex_log_from_real_pos((Real.1).exp)
        (Real.1).exp > Real.0 implies complex_log(Complex.from_real((Real.1).exp)) = Complex.from_real(((Real.1).exp).log.get_or_else(Real.0))
        complex_log(Complex.from_real((Real.1).exp)) = Complex.from_real(((Real.1).exp).log.get_or_else(Real.0))
        ((Real.1).exp).log.get_or_else(Real.0) = Real.1
        Complex.from_real(((Real.1).exp).log.get_or_else(Real.0)) = Complex.from_real(Real.1)
        from_real_one
        Complex.from_real(Real.1) = Complex.1
        Complex.from_real(((Real.1).exp).log.get_or_else(Real.0)) = Complex.1
        complex_log(Complex.from_real((Real.1).exp)) = Complex.1
        w * complex_log(Complex.from_real((Real.1).exp)) = w * Complex.1
        mul_one_right(w)
        w * Complex.1 = w
        complex_exp(w * complex_log(Complex.from_real((Real.1).exp))) = complex_exp(w)
        complex_pow(Complex.from_real((Real.1).exp), w) = complex_exp(w)
    }
}
