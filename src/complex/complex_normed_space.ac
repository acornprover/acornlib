from real import Real
from analysis import real_norm_eq_abs
from complex.complex import Complex, complex_real_smul, mul_from_real
from complex.complex_module import complex_real_module, complex_real_module_smul
from complex.complex_abs import modulus_from_real
from complex.complex_normed import complex_norm_mul
from algebra.module.normed_space import NormedSpace, norm_smul_constraint

/// Real scalar multiplication agrees with multiplying by the embedded real scalar.
/// This bridge lets normed-space proofs use multiplicativity of the complex norm.
theorem complex_real_smul_eq_from_real_mul(r: Real, z: Complex) {
    complex_real_smul(r, z) = Complex.from_real(r) * z
} by {
    mul_from_real(z, r)
    Complex.new(z.re * r, z.im * r) = Complex.new(r * z.re, r * z.im)
}

/// The real embedding is norm-preserving in norm language.
/// This is used in the real-scalar norm compatibility proof and supports the
/// next real-to-complex isometry layer.
theorem complex_from_real_norm(a: Real) {
    Complex.from_real(a).norm = a.norm
} by {
    modulus_from_real(a)
    real_norm_eq_abs(a)
}

/// Norm compatibility for the real scalar action on complex numbers.
theorem complex_real_norm_smul(r: Real, z: Complex) {
    complex_real_smul(r, z).norm = r.norm * z.norm
} by {
    complex_real_smul_eq_from_real_mul(r, z)
    complex_norm_mul(Complex.from_real(r), z)
    complex_from_real_norm(r)
}

/// The real module action on `Complex` satisfies the `NormedSpace` norm-smul constraint.
theorem complex_real_module_norm_smul_constraint {
    norm_smul_constraint[Real, Complex](complex_real_module.smul)
} by {
    norm_smul_constraint[Real, Complex](complex_real_module.smul) = forall(r: Real, z: Complex) {
        complex_real_module.smul(r, z).norm = r.norm * z.norm
    }
    if not norm_smul_constraint[Real, Complex](complex_real_module.smul) {
        let r: Real satisfy {
            exists(z: Complex) {
                complex_real_module.smul(r, z).norm != r.norm * z.norm
            }
        }
        let z: Complex satisfy {
            complex_real_module.smul(r, z).norm != r.norm * z.norm
        }
        complex_real_module_smul(r, z)
        complex_real_norm_smul(r, z)
        false
    }
}

/// The real module on `Complex` satisfies the `NormedSpace` constructor constraint.
/// This is the explicit constrained-constructor bridge used to define
/// `complex_real_normed_space` below.
theorem complex_real_normed_space_constraint {
    NormedSpace.constraint[Real, Complex](complex_real_module)
} by {
    complex_real_module_norm_smul_constraint
}

/// The complex numbers as a normed vector space over the real numbers.
let complex_real_normed_space: NormedSpace[Real, Complex] satisfy {
    NormedSpace.new(complex_real_module) = Option.some(complex_real_normed_space)
}

/// The underlying module of the real normed-space structure on `Complex` is
/// the named complex-as-real module.
theorem complex_real_normed_space_module {
    complex_real_normed_space.module = complex_real_module
} by {
    NormedSpace.new(complex_real_module) = Option.some(complex_real_normed_space)
}
