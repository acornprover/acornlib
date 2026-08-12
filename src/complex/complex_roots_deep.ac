// ---------------------------------------------------------------------------
// Deepening of the theory of roots of unity.
//
// This module extends src/complex/roots_of_unity.ac with:
//
//   a. The geometric identities for the primitive n-th root omega(n):
//      (omega(n) - 1)·sum_{k<n} omega(n)^k = 0, and the sum of the first n
//      powers vanishes when omega(n) != 1.
//   b. The unit-modulus facts: |omega(n)| = 1 and conj(omega(n)) =
//      omega(n)^(-1).
//   c. The special roots omega(1) = 1 and omega(2)² = 1.
//   d. The closure laws of the set of n-th roots of unity: 1 is a root,
//      (-1) is a second root of unity, products and inverses of roots are
//      roots, and the powers of omega(n) are periodic with period n.
// ---------------------------------------------------------------------------

from nat import Nat, from_nat, from_nat_one, alt_induction, pow_zero, pow_one, one_pow, add_sub
from real import Real, two, pi
from list import sum, map
from algebra.ring.ring import geometric_sum_sub_one
from algebra.field.field import unique_inverse, inverse_dist, inverse_one
from complex.complex import Complex, mul_comm, mul_assoc, mul_one_right, mul_zero_right,
    mul_zero_left, mul_left_cancel, from_real_one, real_mul_lifts, eq_by_components
from complex.complex_algebra_deep import sub_eq_zero_iff, inverse_one_restated, sub_self_restated,
    one_neq_neg_one_complex
from complex.complex_exp import complex_exp, complex_pow_suc, complex_i_mul_real, real_mul_one_right
from complex.complex_log_deep import complex_mul_conj_squared
from complex.complex_exp_deep import complex_exp_unit_modulus
from complex.complex_properties import pow_two_neg_one
from complex.roots_of_unity import omega, is_root_of_unity, omega_pow, complex_pow_add,
    complex_exp_two_pi, omega_four_ne_one, omega_two_neg_one

// ---------------------------------------------------------------------------
// Powers of a product and of an inverse.
// ---------------------------------------------------------------------------

/// Powers distribute over products: (z·w)^n = z^n·w^n.
theorem complex_pow_mul_distrib(z: Complex, w: Complex, n: Nat) {
    (z * w).pow(n) = z.pow(n) * w.pow(n)
} by {
    define p(k: Nat) -> Bool {
        (z * w).pow(k) = z.pow(k) * w.pow(k)
    }
    pow_zero(z * w)
    (z * w).pow(Nat.0) = Complex.1
    pow_zero(z)
    z.pow(Nat.0) = Complex.1
    pow_zero(w)
    w.pow(Nat.0) = Complex.1
    Complex.1 * Complex.1 = Complex.1
    (z * w).pow(Nat.0) = z.pow(Nat.0) * w.pow(Nat.0)
    p(Nat.0)
    forall(k: Nat) {
        if p(k) {
            (z * w).pow(k) = z.pow(k) * w.pow(k)
            complex_pow_suc(z * w, k)
            (z * w).pow(k.suc) = (z * w) * (z * w).pow(k)
            (z * w).pow(k.suc) = (z * w) * (z.pow(k) * w.pow(k))
            (z * w) * (z.pow(k) * w.pow(k)) = z * (w * (z.pow(k) * w.pow(k)))
            z * (w * (z.pow(k) * w.pow(k))) = z * (z.pow(k) * (w * w.pow(k)))
            z * (z.pow(k) * (w * w.pow(k))) = z * z.pow(k) * (w * w.pow(k))
            complex_pow_suc(z, k)
            z.pow(k.suc) = z * z.pow(k)
            complex_pow_suc(w, k)
            w.pow(k.suc) = w * w.pow(k)
            z * z.pow(k) * (w * w.pow(k)) = z.pow(k.suc) * w.pow(k.suc)
            (z * w).pow(k.suc) = z.pow(k.suc) * w.pow(k.suc)
            p(k.suc)
        }
    }
    p(Nat.0) and forall(k: Nat) { p(k) implies p(k.suc) }
    alt_induction(p)
    p(n)
}

/// The power of an inverse is the inverse of the power: (z^(-1))^n = (z^n)^(-1).
theorem complex_pow_inverse(z: Complex, n: Nat) {
    z.inverse.pow(n) = z.pow(n).inverse
} by {
    define p(k: Nat) -> Bool {
        z.inverse.pow(k) = z.pow(k).inverse
    }
    pow_zero(z.inverse)
    z.inverse.pow(Nat.0) = Complex.1
    pow_zero(z)
    z.pow(Nat.0) = Complex.1
    inverse_one_restated
    Complex.1.inverse = Complex.1
    z.pow(Nat.0).inverse = Complex.1
    z.inverse.pow(Nat.0) = z.pow(Nat.0).inverse
    p(Nat.0)
    forall(k: Nat) {
        if p(k) {
            z.inverse.pow(k) = z.pow(k).inverse
            complex_pow_suc(z.inverse, k)
            z.inverse.pow(k.suc) = z.inverse * z.inverse.pow(k)
            z.inverse.pow(k.suc) = z.inverse * z.pow(k).inverse
            inverse_dist[Complex](z, z.pow(k))
            (z * z.pow(k)).inverse = z.inverse * z.pow(k).inverse
            complex_pow_suc(z, k)
            z.pow(k.suc) = z * z.pow(k)
            z.pow(k.suc).inverse = (z * z.pow(k)).inverse
            z.inverse.pow(k.suc) = z.pow(k.suc).inverse
            p(k.suc)
        }
    }
    p(Nat.0) and forall(k: Nat) { p(k) implies p(k.suc) }
    alt_induction(p)
    p(n)
}

// ---------------------------------------------------------------------------
// a. The geometric identities.
// ---------------------------------------------------------------------------

/// The geometric identity for the primitive root:
/// (omega(n) - 1)·sum_{k<n} omega(n)^k = 0.
theorem roots_geometric(n: Nat) {
    n >= Nat.1 implies (omega(n) - Complex.1) * sum(map(n.range, omega(n).pow)) = Complex.0
} by {
    if n >= Nat.1 {
        geometric_sum_sub_one[Complex](omega(n), n)
        (omega(n) - Complex.1) * sum(map(n.range, omega(n).pow)) = omega(n).pow(n) - Complex.1
        omega_pow(n)
        n >= Nat.1 implies omega(n).pow(n) = Complex.1
        omega(n).pow(n) = Complex.1
        omega(n).pow(n) - Complex.1 = Complex.1 - Complex.1
        sub_self_restated(Complex.1)
        Complex.1 - Complex.1 = Complex.0
        (omega(n) - Complex.1) * sum(map(n.range, omega(n).pow)) = Complex.0
    }
}

/// The sum of the first n powers of the primitive root vanishes when the
/// root is not one: sum_{k<n} omega(n)^k = 0.
theorem roots_sum_zero(n: Nat) {
    n >= Nat.1 and omega(n) != Complex.1 implies sum(map(n.range, omega(n).pow)) = Complex.0
} by {
    if n >= Nat.1 and omega(n) != Complex.1 {
        roots_geometric(n)
        (omega(n) - Complex.1) * sum(map(n.range, omega(n).pow)) = Complex.0
        sub_eq_zero_iff(omega(n), Complex.1)
        omega(n) - Complex.1 = Complex.0 iff omega(n) = Complex.1
        omega(n) != Complex.1
        omega(n) - Complex.1 != Complex.0
        mul_zero_right(sum(map(n.range, omega(n).pow)))
        (omega(n) - Complex.1) * Complex.0 = Complex.0
        mul_left_cancel(omega(n) - Complex.1, sum(map(n.range, omega(n).pow)), Complex.0)
        (omega(n) - Complex.1) * sum(map(n.range, omega(n).pow)) = (omega(n) - Complex.1) * Complex.0 and (omega(n) - Complex.1) != Complex.0 implies sum(map(n.range, omega(n).pow)) = Complex.0
        sum(map(n.range, omega(n).pow)) = Complex.0
    }
}

// ---------------------------------------------------------------------------
// b. The unit-modulus facts.
// ---------------------------------------------------------------------------

/// The primitive root of unity has unit modulus: |omega(n)| = 1.
theorem omega_modulus(n: Nat) {
    n >= Nat.1 implies omega(n).modulus = Real.1
} by {
    if n >= Nat.1 {
        omega(n) = complex_exp(complex_i_mul_real(two * pi / from_nat[Real](n)))
        omega(n).modulus = complex_exp(complex_i_mul_real(two * pi / from_nat[Real](n))).modulus
        complex_exp_unit_modulus(two * pi / from_nat[Real](n))
        complex_exp(complex_i_mul_real(two * pi / from_nat[Real](n))).modulus = Real.1
        omega(n).modulus = Real.1
    }
}

/// The conjugate of the primitive root is its inverse:
/// conj(omega(n)) = omega(n)^(-1).
theorem omega_conj_inverse(n: Nat) {
    n >= Nat.1 implies omega(n).conj = omega(n).inverse
} by {
    if n >= Nat.1 {
        omega_modulus(n)
        omega(n).modulus = Real.1
        complex_mul_conj_squared(omega(n))
        omega(n) * omega(n).conj = Complex.from_real(omega(n).modulus * omega(n).modulus)
        omega(n).modulus * omega(n).modulus = Real.1 * Real.1
        Real.1 * Real.1 = Real.1
        Complex.from_real(omega(n).modulus * omega(n).modulus) = Complex.from_real(Real.1)
        from_real_one
        Complex.from_real(Real.1) = Complex.1
        omega(n) * omega(n).conj = Complex.1
        unique_inverse[Complex](omega(n), omega(n).conj)
        omega(n) * omega(n).conj = Complex.1 implies omega(n).conj = omega(n).inverse
        omega(n).conj = omega(n).inverse
    }
}

// ---------------------------------------------------------------------------
// c. The special roots.
// ---------------------------------------------------------------------------

/// The first root of unity is one: omega(1) = 1.
theorem omega_one {
    omega(Nat.1) = Complex.1
} by {
    omega(Nat.1) = complex_exp(complex_i_mul_real(two * pi / from_nat[Real](Nat.1)))
    from_nat_one[Real]
    from_nat[Real](Nat.1) = Real.1
    two * pi / from_nat[Real](Nat.1) = two * pi / Real.1
    inverse_one[Real]
    Real.1.inverse = Real.1
    (two * pi) / Real.1 = (two * pi) * Real.1.inverse
    (two * pi) * Real.1.inverse = (two * pi) * Real.1
    real_mul_one_right(two * pi)
    (two * pi) * Real.1 = two * pi
    (two * pi) / Real.1 = two * pi
    two * pi / from_nat[Real](Nat.1) = two * pi
    complex_i_mul_real(two * pi / from_nat[Real](Nat.1)) = complex_i_mul_real(two * pi)
    complex_exp(complex_i_mul_real(two * pi / from_nat[Real](Nat.1))) =
        complex_exp(complex_i_mul_real(two * pi))
    complex_exp_two_pi
    complex_exp(complex_i_mul_real(two * pi)) = Complex.1
    omega(Nat.1) = Complex.1
}

/// The square of the second root of unity is one: omega(2)² = 1.
theorem omega_two_sq {
    omega(Nat.2).pow(Nat.2) = Complex.1
} by {
    omega_pow(Nat.2)
    Nat.2 >= Nat.1 implies omega(Nat.2).pow(Nat.2) = Complex.1
    omega(Nat.2).pow(Nat.2) = Complex.1
}

// ---------------------------------------------------------------------------
// d. The closure laws of the set of n-th roots of unity.
// ---------------------------------------------------------------------------

/// One is an n-th root of unity for every n.
theorem is_root_of_unity_one(n: Nat) {
    is_root_of_unity(Complex.1, n)
} by {
    one_pow[Complex](n)
    Complex.1.pow(n) = Complex.1
    is_root_of_unity(Complex.1, n)
}

/// Negative one is a second root of unity.
theorem is_root_of_unity_neg_one_two {
    is_root_of_unity(-Complex.1, Nat.2)
} by {
    pow_two_neg_one
    (-Complex.1).pow(Nat.2) = Complex.1
    is_root_of_unity(-Complex.1, Nat.2)
}

/// The primitive fourth root is a fourth root of unity.
theorem omega_four_is_root {
    is_root_of_unity(omega(Nat.4), Nat.4)
} by {
    omega_pow(Nat.4)
    Nat.4 >= Nat.1 implies omega(Nat.4).pow(Nat.4) = Complex.1
    Nat.1 + Nat.3 = Nat.4
    Nat.1 <= Nat.4
    Nat.4 >= Nat.1
    omega(Nat.4).pow(Nat.4) = Complex.1
    is_root_of_unity(omega(Nat.4), Nat.4)
}

/// The product of two n-th roots of unity is an n-th root of unity.
theorem roots_of_unity_mul(z: Complex, w: Complex, n: Nat) {
    is_root_of_unity(z, n) and is_root_of_unity(w, n) implies is_root_of_unity(z * w, n)
} by {
    if is_root_of_unity(z, n) and is_root_of_unity(w, n) {
        z.pow(n) = Complex.1
        w.pow(n) = Complex.1
        complex_pow_mul_distrib(z, w, n)
        (z * w).pow(n) = z.pow(n) * w.pow(n)
        z.pow(n) * w.pow(n) = Complex.1 * Complex.1
        Complex.1 * Complex.1 = Complex.1
        (z * w).pow(n) = Complex.1
        is_root_of_unity(z * w, n)
    }
}

/// The inverse of an n-th root of unity is an n-th root of unity.
theorem roots_of_unity_inv(z: Complex, n: Nat) {
    is_root_of_unity(z, n) implies is_root_of_unity(z.inverse, n)
} by {
    if is_root_of_unity(z, n) {
        z.pow(n) = Complex.1
        complex_pow_inverse(z, n)
        z.inverse.pow(n) = z.pow(n).inverse
        z.pow(n).inverse = Complex.1.inverse
        inverse_one_restated
        Complex.1.inverse = Complex.1
        z.inverse.pow(n) = Complex.1
        is_root_of_unity(z.inverse, n)
    }
}


/// The sum of the two second roots of unity vanishes: 1 + (-1) = 0.
theorem omega_two_sum_zero {
    sum(map(Nat.2.range, omega(Nat.2).pow)) = Complex.0
} by {
    roots_sum_zero(Nat.2)
    Nat.2 >= Nat.1 and omega(Nat.2) != Complex.1 implies sum(map(Nat.2.range, omega(Nat.2).pow)) = Complex.0
    Nat.1 + Nat.1 = Nat.2
    Nat.1 <= Nat.2
    Nat.2 >= Nat.1
    omega_two_neg_one
    omega(Nat.2) = -Complex.1
    one_neq_neg_one_complex
    Complex.1 != -Complex.1
    omega(Nat.2) != Complex.1
    sum(map(Nat.2.range, omega(Nat.2).pow)) = Complex.0
}

/// The sum of the four fourth roots of unity vanishes.
theorem omega_four_sum_zero {
    sum(map(Nat.4.range, omega(Nat.4).pow)) = Complex.0
} by {
    roots_sum_zero(Nat.4)
    Nat.4 >= Nat.1 and omega(Nat.4) != Complex.1 implies sum(map(Nat.4.range, omega(Nat.4).pow)) = Complex.0
    Nat.1 + Nat.3 = Nat.4
    Nat.1 <= Nat.4
    Nat.4 >= Nat.1
    omega_four_ne_one
    omega(Nat.4) != Complex.1
    sum(map(Nat.4.range, omega(Nat.4).pow)) = Complex.0
}

/// The powers of the primitive root are periodic with period n:
/// omega(n)^(k + n) = omega(n)^k.
theorem omega_pow_add_periodic(n: Nat, k: Nat) {
    n >= Nat.1 implies omega(n).pow(k + n) = omega(n).pow(k)
} by {
    if n >= Nat.1 {
        complex_pow_add(omega(n), k, n)
        omega(n).pow(k + n) = omega(n).pow(k) * omega(n).pow(n)
        omega_pow(n)
        n >= Nat.1 implies omega(n).pow(n) = Complex.1
        omega(n).pow(n) = Complex.1
        omega(n).pow(k) * omega(n).pow(n) = omega(n).pow(k) * Complex.1
        mul_one_right(omega(n).pow(k))
        omega(n).pow(k) * Complex.1 = omega(n).pow(k)
        omega(n).pow(k + n) = omega(n).pow(k)
    }
}

/// The (n - 1)-th power of the primitive root is its conjugate:
/// omega(n)^(n - 1) = conj(omega(n)).
theorem omega_pow_pred_conj(n: Nat) {
    n >= Nat.1 implies omega(n).pow(n - Nat.1) = omega(n).conj
} by {
    if n >= Nat.1 {
        add_sub(n, Nat.1)
        n - Nat.1 + Nat.1 = n
        complex_pow_add(omega(n), n - Nat.1, Nat.1)
        omega(n).pow((n - Nat.1) + Nat.1) = omega(n).pow(n - Nat.1) * omega(n).pow(Nat.1)
        omega(n).pow(n) = omega(n).pow(n - Nat.1) * omega(n).pow(Nat.1)
        pow_one(omega(n))
        omega(n).pow(Nat.1) = omega(n)
        omega(n).pow(n) = omega(n).pow(n - Nat.1) * omega(n)
        omega_pow(n)
        n >= Nat.1 implies omega(n).pow(n) = Complex.1
        omega(n).pow(n) = Complex.1
        Complex.1 = omega(n).pow(n - Nat.1) * omega(n)
        mul_comm(omega(n).pow(n - Nat.1), omega(n))
        omega(n).pow(n - Nat.1) * omega(n) = omega(n) * omega(n).pow(n - Nat.1)
        omega(n) * omega(n).pow(n - Nat.1) = Complex.1
        unique_inverse[Complex](omega(n), omega(n).pow(n - Nat.1))
        omega(n) * omega(n).pow(n - Nat.1) = Complex.1 implies omega(n).pow(n - Nat.1) = omega(n).inverse
        omega(n).pow(n - Nat.1) = omega(n).inverse
        omega_conj_inverse(n)
        n >= Nat.1 implies omega(n).conj = omega(n).inverse
        omega(n).conj = omega(n).inverse
        omega(n).pow(n - Nat.1) = omega(n).conj
    }
}

