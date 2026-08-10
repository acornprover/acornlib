from real import Real, two, pi
from nat import Nat, pow_one, one_pow, pow_pow
from algebra.ring.ring import mul_neg_neg, neg_one_mul_neg_one
from complex.complex import Complex, conj_mul, conj_add, abs_squared_conj, re_from_real,
    i_squared_eq_neg_one, add_zero_left, mul_one_right
from complex.complex_abs import modulus_conj, modulus_squared
from complex.complex_exp import complex_exp, complex_exp_add, complex_i_mul_real,
    complex_pow_suc, euler_identity
from complex.roots_of_unity import omega, is_root_of_unity, omega_pow_is_root_of_unity,
    complex_exp_two_pi

// ---------------------------------------------------------------------------
// Complex analysis properties.
//
// This file collects the fundamental algebraic properties of the complex
// numbers that are used throughout complex analysis:
//
//   a. Conjugation is a ring homomorphism: conj(z·w) = conj(z)·conj(w) and
//      conj(z + w) = conj(z) + conj(w).
//   b. The fundamental identity |z|² = z·conj(z).
//   c. The roots of unity: z^n = 1 has the solutions e^(2·pi·i·k/n) for
//      k = 0..n-1; verified for n = 4 with the roots 1, i, -1, -i.
//   d. Conjugation preserves the modulus: |conj(z)| = |z|.
//   e. The exponential is periodic: exp(z + 2·pi·i) = exp(z); at z = 0 this
//      is Euler's identity exp(2·pi·i) = 1.
// ---------------------------------------------------------------------------

// ---------------------------------------------------------------------------
// a. The algebra of conjugation.
// ---------------------------------------------------------------------------

/// The conjugate of a product is the product of the conjugates:
/// conj(z·w) = conj(z)·conj(w).
theorem conj_product(a: Complex, b: Complex) {
    (a * b).conj = a.conj * b.conj
} by {
    conj_mul(a, b)
    (a * b).conj = a.conj * b.conj
}

/// The conjugate of a sum is the sum of the conjugates:
/// conj(z + w) = conj(z) + conj(w).
theorem conj_sum(a: Complex, b: Complex) {
    (a + b).conj = a.conj + b.conj
} by {
    conj_add(a, b)
    (a + b).conj = a.conj + b.conj
}

// ---------------------------------------------------------------------------
// b. The fundamental identity |z|² = z·conj(z).
// ---------------------------------------------------------------------------

/// The product of a complex number with its conjugate is its squared modulus:
/// z·conj(z) = |z|².
theorem mul_conj_modulus_squared(z: Complex) {
    z * z.conj = Complex.from_real(z.modulus * z.modulus)
} by {
    abs_squared_conj(z)
    z * z.conj = Complex.new(z.abs_squared, Real.0)
    modulus_squared(z)
    z.modulus * z.modulus = z.abs_squared
    Complex.from_real(z.abs_squared) = Complex.new(z.abs_squared, Real.0)
    z * z.conj = Complex.from_real(z.modulus * z.modulus)
}

/// The real part of z·conj(z) is the squared modulus.
theorem re_mul_conj_modulus_squared(z: Complex) {
    (z * z.conj).re = z.modulus * z.modulus
} by {
    mul_conj_modulus_squared(z)
    z * z.conj = Complex.from_real(z.modulus * z.modulus)
    re_from_real(z.modulus * z.modulus)
    Complex.from_real(z.modulus * z.modulus).re = z.modulus * z.modulus
    (z * z.conj).re = z.modulus * z.modulus
}

// ---------------------------------------------------------------------------
// c. The roots of unity.
// ---------------------------------------------------------------------------

/// The roots of unity: for every k < n, the power omega_n^k is an n-th root
/// of unity, i.e. (e^(2·pi·i/n))^k is a solution of z^n = 1.
theorem roots_of_unity_solution(n: Nat, k: Nat) {
    n >= Nat.1 and k < n implies is_root_of_unity(omega(n).pow(k), n)
} by {
    if n >= Nat.1 and k < n {
        omega_pow_is_root_of_unity(n, k)
        n >= Nat.1 implies is_root_of_unity(omega(n).pow(k), n)
        is_root_of_unity(omega(n).pow(k), n)
    }
}

/// The square of the imaginary unit is negative one: i² = -1.
theorem pow_two_i {
    Complex.i.pow(Nat.2) = -Complex.1
} by {
    complex_pow_suc(Complex.i, Nat.1)
    Complex.i.pow(Nat.1.suc) = Complex.i * Complex.i.pow(Nat.1)
    Nat.1.suc = Nat.2
    Complex.i.pow(Nat.2) = Complex.i * Complex.i.pow(Nat.1)
    pow_one(Complex.i)
    Complex.i.pow(Nat.1) = Complex.i
    Complex.i.pow(Nat.2) = Complex.i * Complex.i
    i_squared_eq_neg_one
    Complex.i * Complex.i = -Complex.1
    Complex.i.pow(Nat.2) = -Complex.1
}

/// The square of negative one is one: (-1)² = 1.
theorem pow_two_neg_one {
    (-Complex.1).pow(Nat.2) = Complex.1
} by {
    complex_pow_suc(-Complex.1, Nat.1)
    (-Complex.1).pow(Nat.1.suc) = (-Complex.1) * (-Complex.1).pow(Nat.1)
    Nat.1.suc = Nat.2
    (-Complex.1).pow(Nat.2) = (-Complex.1) * (-Complex.1).pow(Nat.1)
    pow_one(-Complex.1)
    (-Complex.1).pow(Nat.1) = -Complex.1
    (-Complex.1).pow(Nat.2) = (-Complex.1) * (-Complex.1)
    neg_one_mul_neg_one[Complex]
    -Complex.1 * -Complex.1 = Complex.1
    (-Complex.1).pow(Nat.2) = Complex.1
}

/// Negation does not change the square: (-z)² = z².
theorem pow_two_neg(a: Complex) {
    (-a).pow(Nat.2) = a.pow(Nat.2)
} by {
    complex_pow_suc(-a, Nat.1)
    (-a).pow(Nat.1.suc) = (-a) * (-a).pow(Nat.1)
    Nat.1.suc = Nat.2
    (-a).pow(Nat.2) = (-a) * (-a).pow(Nat.1)
    pow_one(-a)
    (-a).pow(Nat.1) = -a
    (-a).pow(Nat.2) = (-a) * (-a)
    complex_pow_suc(a, Nat.1)
    a.pow(Nat.1.suc) = a * a.pow(Nat.1)
    a.pow(Nat.2) = a * a.pow(Nat.1)
    pow_one(a)
    a.pow(Nat.1) = a
    a.pow(Nat.2) = a * a
    mul_neg_neg[Complex](a, a)
    -a * -a = a * a
    (-a).pow(Nat.2) = a.pow(Nat.2)
}

/// One is a fourth root of unity: 1⁴ = 1.
theorem pow_four_one {
    Complex.1.pow(Nat.4) = Complex.1
} by {
    one_pow[Complex](Nat.4)
    Complex.1.pow(Nat.4) = Complex.1
}

/// The imaginary unit is a fourth root of unity: i⁴ = 1.
theorem pow_four_i {
    Complex.i.pow(Nat.4) = Complex.1
} by {
    pow_two_i
    Complex.i.pow(Nat.2) = -Complex.1
    pow_pow[Complex](Complex.i, Nat.2, Nat.2)
    Complex.i.pow(Nat.2).pow(Nat.2) = Complex.i.pow(Nat.2 * Nat.2)
    Nat.2 * Nat.2 = Nat.4
    Complex.i.pow(Nat.2 * Nat.2) = Complex.i.pow(Nat.4)
    Complex.i.pow(Nat.4) = Complex.i.pow(Nat.2).pow(Nat.2)
    Complex.i.pow(Nat.4) = (-Complex.1).pow(Nat.2)
    pow_two_neg_one
    (-Complex.1).pow(Nat.2) = Complex.1
    Complex.i.pow(Nat.4) = Complex.1
}

/// Negative one is a fourth root of unity: (-1)⁴ = 1.
theorem pow_four_neg_one {
    (-Complex.1).pow(Nat.4) = Complex.1
} by {
    pow_pow[Complex](-Complex.1, Nat.2, Nat.2)
    (-Complex.1).pow(Nat.2).pow(Nat.2) = (-Complex.1).pow(Nat.2 * Nat.2)
    Nat.2 * Nat.2 = Nat.4
    (-Complex.1).pow(Nat.2 * Nat.2) = (-Complex.1).pow(Nat.4)
    (-Complex.1).pow(Nat.4) = (-Complex.1).pow(Nat.2).pow(Nat.2)
    pow_two_neg_one
    (-Complex.1).pow(Nat.2) = Complex.1
    (-Complex.1).pow(Nat.4) = Complex.1.pow(Nat.2)
    one_pow[Complex](Nat.2)
    Complex.1.pow(Nat.2) = Complex.1
    (-Complex.1).pow(Nat.4) = Complex.1
}

/// Negative the imaginary unit is a fourth root of unity: (-i)⁴ = 1.
theorem pow_four_neg_i {
    (-Complex.i).pow(Nat.4) = Complex.1
} by {
    pow_two_neg(Complex.i)
    (-Complex.i).pow(Nat.2) = Complex.i.pow(Nat.2)
    pow_two_i
    Complex.i.pow(Nat.2) = -Complex.1
    (-Complex.i).pow(Nat.2) = -Complex.1
    pow_pow[Complex](-Complex.i, Nat.2, Nat.2)
    (-Complex.i).pow(Nat.2).pow(Nat.2) = (-Complex.i).pow(Nat.2 * Nat.2)
    Nat.2 * Nat.2 = Nat.4
    (-Complex.i).pow(Nat.2 * Nat.2) = (-Complex.i).pow(Nat.4)
    (-Complex.i).pow(Nat.4) = (-Complex.i).pow(Nat.2).pow(Nat.2)
    (-Complex.i).pow(Nat.4) = (-Complex.1).pow(Nat.2)
    pow_two_neg_one
    (-Complex.1).pow(Nat.2) = Complex.1
    (-Complex.i).pow(Nat.4) = Complex.1
}

/// A fourth root of unity: a complex number z with z⁴ = 1.
define is_fourth_root_of_unity(z: Complex) -> Bool {
    z.pow(Nat.4) = Complex.1
}

/// The four numbers 1, i, -1, -i are fourth roots of unity: each satisfies
/// z⁴ = 1.
theorem four_fourth_roots {
    is_fourth_root_of_unity(Complex.1) and
        is_fourth_root_of_unity(Complex.i) and
        is_fourth_root_of_unity(-Complex.1) and
        is_fourth_root_of_unity(-Complex.i)
} by {
    pow_four_one
    Complex.1.pow(Nat.4) = Complex.1
    is_fourth_root_of_unity(Complex.1)
    pow_four_i
    Complex.i.pow(Nat.4) = Complex.1
    is_fourth_root_of_unity(Complex.i)
    pow_four_neg_one
    (-Complex.1).pow(Nat.4) = Complex.1
    is_fourth_root_of_unity(-Complex.1)
    pow_four_neg_i
    (-Complex.i).pow(Nat.4) = Complex.1
    is_fourth_root_of_unity(-Complex.i)
}

/// Every power of the primitive fourth root of unity is a fourth root of
/// unity: (omega_4)^k solves z⁴ = 1 for every k.
theorem fourth_roots_from_omega(k: Nat) {
    is_root_of_unity(omega(Nat.4).pow(k), Nat.4)
} by {
    omega_pow_is_root_of_unity(Nat.4, k)
    Nat.4 >= Nat.1 implies is_root_of_unity(omega(Nat.4).pow(k), Nat.4)
    Nat.1 + Nat.3 = Nat.4
    Nat.1 <= Nat.4
    Nat.4 >= Nat.1
    is_root_of_unity(omega(Nat.4).pow(k), Nat.4)
}

// ---------------------------------------------------------------------------
// d. Conjugation preserves the modulus.
// ---------------------------------------------------------------------------

/// Conjugation preserves the modulus: |conj(z)| = |z|.
theorem modulus_of_conj(z: Complex) {
    z.conj.modulus = z.modulus
} by {
    modulus_conj(z)
    z.conj.modulus = z.modulus
}

// ---------------------------------------------------------------------------
// e. The exponential is periodic.
// ---------------------------------------------------------------------------

/// The complex exponential has fundamental period 2·pi·i:
/// exp(z + 2·pi·i) = exp(z).
theorem complex_exp_periodic(z: Complex) {
    complex_exp(z + complex_i_mul_real(two * pi)) = complex_exp(z)
} by {
    complex_exp_add(z, complex_i_mul_real(two * pi))
    complex_exp(z + complex_i_mul_real(two * pi)) =
        complex_exp(z) * complex_exp(complex_i_mul_real(two * pi))
    complex_exp_two_pi
    complex_exp(complex_i_mul_real(two * pi)) = Complex.1
    mul_one_right(complex_exp(z))
    complex_exp(z) * Complex.1 = complex_exp(z)
    complex_exp(z + complex_i_mul_real(two * pi)) = complex_exp(z)
}

/// Euler's identity at 2·pi·i: exp(2·pi·i) = 1.
theorem complex_exp_two_pi_i {
    complex_exp(complex_i_mul_real(two * pi)) = Complex.1
} by {
    complex_exp_two_pi
    complex_exp(complex_i_mul_real(two * pi)) = Complex.1
}

/// The period evaluated at zero: exp(0 + 2·pi·i) = 1.
theorem complex_exp_zero_period {
    complex_exp(Complex.0 + complex_i_mul_real(two * pi)) = Complex.1
} by {
    add_zero_left(complex_i_mul_real(two * pi))
    Complex.0 + complex_i_mul_real(two * pi) = complex_i_mul_real(two * pi)
    complex_exp_two_pi
    complex_exp(complex_i_mul_real(two * pi)) = Complex.1
    complex_exp(Complex.0 + complex_i_mul_real(two * pi)) = Complex.1
}

/// Euler's identity: exp(i·pi) = -1.
theorem euler_identity_restated {
    complex_exp(Complex.i * Complex.from_real(pi)) = -Complex.1
} by {
    euler_identity
    complex_exp(Complex.i * Complex.from_real(pi)) = -Complex.1
}
