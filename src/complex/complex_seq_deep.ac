// ---------------------------------------------------------------------------
// Deepening of the theory of complex sequences.
//
// This module extends src/complex/complex_seq.ac with the interaction between
// convergence and the modulus, and with the convergence of powers:
//
//   a. The modulus of a convergent complex sequence converges to the modulus
//      of the limit: |z(n)| -> |w| whenever z(n) -> w.  The proof passes
//      through the induced metric and the reverse triangle inequality.
//   b. Consequently |z(n)|² -> |w|², and the limit of the modulus sequence
//      is the modulus of the complex limit.
//   c. Powers converge: if z(n) -> w then z(n)² -> w², z(n)³ -> w³, and by
//      induction z(n)^k -> w^k for every k.
//   d. A convergent complex sequence is eventually bounded.
// ---------------------------------------------------------------------------

from real import Real, converges, converges_to, limit, tail_bound, converges_to_imp_converges,
    converges_imp_converges_to, converges_to_unique, prod_seq, limit_prod_seq, lt_add_right,
    lte_abs, add_comm
from order import lt_imp_lte, lte_lt_trans
from nat import Nat, alt_induction, pow_zero, one_pow
from analysis import tendsto_metric, metric_tail_bound, metric_tail_bound_at,
    tendsto_metric_tail_bound
from complex.complex import Complex
from complex.complex_abs import modulus_reverse_triangle, modulus_neg
from complex.complex_abs_deep import modulus_le_add
from complex.complex_metric import complex_converges_to_imp_tendsto_metric,
    complex_distance_eq_modulus_sub
from complex.complex_seq import complex_converges_to, complex_converges, complex_limit,
    complex_mul_seq, const_seq, const_seq_converges_to, complex_converges_imp_converges_to_limit,
    complex_converges_to_imp_limit, complex_limit_of_mul_seq, complex_mul_seq_converges_to,
    complex_converges_to_imp_converges
from complex.complex_exp import complex_pow_suc, real_one_positive

/// The sequence of moduli of a complex sequence.
define complex_modulus_seq(z: Nat -> Complex, n: Nat) -> Real {
    z(n).modulus
}

/// The pointwise k-th powers of a complex sequence.
define complex_pow_seq(z: Nat -> Complex, k: Nat, n: Nat) -> Complex {
    z(n).pow(k)
}

// ---------------------------------------------------------------------------
// a. The modulus of a convergent sequence.
// ---------------------------------------------------------------------------

/// The modulus of a convergent complex sequence converges to the modulus of
/// the limit: |z(n)| -> |w|.
theorem complex_seq_abs_converges_to(z: Nat -> Complex, w: Complex) {
    complex_converges_to(z, w) implies converges_to(complex_modulus_seq(z), w.modulus)
} by {
    if complex_converges_to(z, w) {
        complex_converges_to_imp_tendsto_metric(z, w)
        tendsto_metric(z, w)
        forall(eps: Real) {
            if eps.is_positive {
                tendsto_metric_tail_bound(z, w, eps)
                tendsto_metric(z, w) and eps.is_positive implies exists(n: Nat) {
                    metric_tail_bound(z, w, n, eps)
                }
                exists(n: Nat) {
                    metric_tail_bound(z, w, n, eps)
                }
                let n: Nat satisfy {
                    metric_tail_bound(z, w, n, eps)
                }
                forall(i: Nat) {
                    if n <= i {
                        metric_tail_bound_at(z, w, n, eps, i)
                        metric_tail_bound(z, w, n, eps) and n <= i implies z(i).distance(w) < eps
                        z(i).distance(w) < eps
                        complex_distance_eq_modulus_sub(z(i), w)
                        z(i).distance(w) = (z(i) - w).modulus
                        (z(i) - w).modulus < eps
                        modulus_reverse_triangle(z(i), w)
                        (z(i).modulus - w.modulus).abs <= (z(i) - w).modulus
                        lte_lt_trans((z(i).modulus - w.modulus).abs, (z(i) - w).modulus, eps)
                        (z(i).modulus - w.modulus).abs < eps
                        complex_modulus_seq(z, i) = z(i).modulus
                        complex_modulus_seq(z, i).is_close(w.modulus, eps) = (complex_modulus_seq(z, i) - w.modulus).abs < eps
                        (complex_modulus_seq(z, i) - w.modulus).abs < eps
                        complex_modulus_seq(z, i).is_close(w.modulus, eps)
                    }
                }
                tail_bound(complex_modulus_seq(z), w.modulus, n, eps)
                exists(m: Nat) {
                    tail_bound(complex_modulus_seq(z), w.modulus, m, eps)
                }
            }
        }
        converges_to(complex_modulus_seq(z), w.modulus) = forall(eps: Real) {
            eps.is_positive implies exists(n: Nat) {
                tail_bound(complex_modulus_seq(z), w.modulus, n, eps)
            }
        }
        if not converges_to(complex_modulus_seq(z), w.modulus) {
            not forall(eps: Real) {
                eps.is_positive implies exists(n: Nat) {
                    tail_bound(complex_modulus_seq(z), w.modulus, n, eps)
                }
            }
            let bad_eps: Real satisfy {
                bad_eps.is_positive and forall(n: Nat) {
                    not tail_bound(complex_modulus_seq(z), w.modulus, n, bad_eps)
                }
            }
            bad_eps.is_positive
            forall(n: Nat) {
                not tail_bound(complex_modulus_seq(z), w.modulus, n, bad_eps)
            }
            bad_eps.is_positive implies exists(n: Nat) {
                tail_bound(complex_modulus_seq(z), w.modulus, n, bad_eps)
            }
            exists(n: Nat) {
                tail_bound(complex_modulus_seq(z), w.modulus, n, bad_eps)
            }
            let m0: Nat satisfy {
                tail_bound(complex_modulus_seq(z), w.modulus, m0, bad_eps)
            }
            not tail_bound(complex_modulus_seq(z), w.modulus, m0, bad_eps)
            false
        }
        converges_to(complex_modulus_seq(z), w.modulus)
    }
}

/// The modulus of a convergent complex sequence converges.
theorem complex_seq_abs_converges(z: Nat -> Complex) {
    complex_converges(z) implies converges(complex_modulus_seq(z))
} by {
    if complex_converges(z) {
        let w: Complex satisfy {
            complex_converges_to(z, w)
        }
        complex_seq_abs_converges_to(z, w)
        converges_to(complex_modulus_seq(z), w.modulus)
        converges_to_imp_converges(complex_modulus_seq(z), w.modulus)
        converges(complex_modulus_seq(z))
    }
}

/// The limit of the modulus sequence is the modulus of the complex limit.
theorem complex_seq_modulus_limit(z: Nat -> Complex) {
    complex_converges(z) implies limit(complex_modulus_seq(z)) = complex_limit(z).modulus
} by {
    if complex_converges(z) {
        complex_converges_imp_converges_to_limit(z)
        complex_converges_to(z, complex_limit(z))
        complex_seq_abs_converges_to(z, complex_limit(z))
        converges_to(complex_modulus_seq(z), complex_limit(z).modulus)
        converges_to_imp_converges(complex_modulus_seq(z), complex_limit(z).modulus)
        converges(complex_modulus_seq(z))
        converges_imp_converges_to(complex_modulus_seq(z))
        converges_to(complex_modulus_seq(z), limit(complex_modulus_seq(z)))
        converges_to_unique(complex_modulus_seq(z), complex_limit(z).modulus, limit(complex_modulus_seq(z)))
        limit(complex_modulus_seq(z)) = complex_limit(z).modulus
    }
}

// ---------------------------------------------------------------------------
// b. The squared modulus.
// ---------------------------------------------------------------------------

/// The squared modulus of a convergent sequence converges to the squared
/// modulus of the limit: |z(n)|² -> |w|².
theorem complex_seq_abs_sq_converges_to(z: Nat -> Complex, w: Complex) {
    complex_converges_to(z, w) implies converges_to(prod_seq(complex_modulus_seq(z), complex_modulus_seq(z)), w.modulus * w.modulus)
} by {
    if complex_converges_to(z, w) {
        complex_seq_abs_converges_to(z, w)
        converges_to(complex_modulus_seq(z), w.modulus)
        converges_to_imp_converges(complex_modulus_seq(z), w.modulus)
        converges(complex_modulus_seq(z))
        limit_prod_seq(complex_modulus_seq(z), complex_modulus_seq(z))
        converges(complex_modulus_seq(z)) and converges(complex_modulus_seq(z)) implies converges_to(prod_seq(complex_modulus_seq(z), complex_modulus_seq(z)),
                limit(complex_modulus_seq(z)) * limit(complex_modulus_seq(z)))
        converges_to(prod_seq(complex_modulus_seq(z), complex_modulus_seq(z)),
            limit(complex_modulus_seq(z)) * limit(complex_modulus_seq(z)))
        converges_imp_converges_to(complex_modulus_seq(z))
        converges_to(complex_modulus_seq(z), limit(complex_modulus_seq(z)))
        converges_to_unique(complex_modulus_seq(z), w.modulus, limit(complex_modulus_seq(z)))
        limit(complex_modulus_seq(z)) = w.modulus
        converges_to(prod_seq(complex_modulus_seq(z), complex_modulus_seq(z)), w.modulus * w.modulus)
    }
}

// ---------------------------------------------------------------------------
// c. Powers converge.
// ---------------------------------------------------------------------------

/// Squares converge: if z(n) -> w then z(n)² -> w².
theorem complex_seq_square_converges_to(z: Nat -> Complex, w: Complex) {
    complex_converges_to(z, w) implies complex_converges_to(complex_mul_seq(z, z), w * w)
} by {
    if complex_converges_to(z, w) {
        complex_mul_seq_converges_to(z, z, w, w)
        complex_converges_to(complex_mul_seq(z, z), w * w)
    }
}

/// The limit of the squares is the square of the limit.
theorem complex_seq_square_limit(z: Nat -> Complex) {
    complex_converges(z) implies complex_limit(complex_mul_seq(z, z)) = complex_limit(z) * complex_limit(z)
} by {
    if complex_converges(z) {
        complex_limit_of_mul_seq(z, z)
        complex_converges(z) and complex_converges(z) implies complex_limit(complex_mul_seq(z, z)) = complex_limit(z) * complex_limit(z)
        complex_limit(complex_mul_seq(z, z)) = complex_limit(z) * complex_limit(z)
    }
}

/// The pointwise powers at exponent zero are the constant one sequence.
theorem complex_pow_seq_zero_eq(z: Nat -> Complex) {
    complex_pow_seq(z, Nat.0) = const_seq(Complex.1)
} by {
    forall(i: Nat) {
        complex_pow_seq(z, Nat.0, i) = z(i).pow(Nat.0)
        pow_zero(z(i))
        z(i).pow(Nat.0) = Complex.1
        const_seq(Complex.1, i) = Complex.1
        complex_pow_seq(z, Nat.0, i) = const_seq(Complex.1, i)
    }
    complex_pow_seq(z, Nat.0) = const_seq(Complex.1)
}

/// The pointwise powers at exponent m + 1 are the pointwise product of the
/// sequence with its m-th powers.
theorem complex_pow_seq_suc_eq(z: Nat -> Complex, m: Nat) {
    complex_pow_seq(z, m.suc) = complex_mul_seq(z, complex_pow_seq(z, m))
} by {
    forall(i: Nat) {
        complex_pow_seq(z, m.suc, i) = z(i).pow(m.suc)
        complex_pow_suc(z(i), m)
        z(i).pow(m.suc) = z(i) * z(i).pow(m)
        complex_pow_seq(z, m.suc, i) = z(i) * complex_pow_seq(z, m, i)
        complex_mul_seq(z, complex_pow_seq(z, m), i) = z(i) * complex_pow_seq(z, m, i)
        complex_pow_seq(z, m.suc, i) = complex_mul_seq(z, complex_pow_seq(z, m), i)
    }
    complex_pow_seq(z, m.suc) = complex_mul_seq(z, complex_pow_seq(z, m))
}

/// Powers converge: if z(n) -> w then z(n)^k -> w^k for every k.
theorem complex_seq_pow_converges_to(z: Nat -> Complex, w: Complex, k: Nat) {
    complex_converges_to(z, w) implies complex_converges_to(complex_pow_seq(z, k), w.pow(k))
} by {
    define p(m: Nat) -> Bool {
        complex_converges_to(complex_pow_seq(z, m), w.pow(m))
    }
    if complex_converges_to(z, w) {
        complex_pow_seq_zero_eq(z)
        complex_pow_seq(z, Nat.0) = const_seq(Complex.1)
        const_seq_converges_to(Complex.1)
        complex_converges_to(const_seq(Complex.1), Complex.1)
        complex_converges_to(complex_pow_seq(z, Nat.0), Complex.1)
        pow_zero(w)
        w.pow(Nat.0) = Complex.1
        complex_converges_to(complex_pow_seq(z, Nat.0), w.pow(Nat.0))
        p(Nat.0)
        forall(m: Nat) {
            if p(m) {
                complex_converges_to(complex_pow_seq(z, m), w.pow(m))
                complex_pow_seq_suc_eq(z, m)
                complex_pow_seq(z, m.suc) = complex_mul_seq(z, complex_pow_seq(z, m))
                complex_mul_seq_converges_to(z, complex_pow_seq(z, m), w, w.pow(m))
                complex_converges_to(complex_mul_seq(z, complex_pow_seq(z, m)), w * w.pow(m))
                complex_converges_to(complex_pow_seq(z, m.suc), w * w.pow(m))
                complex_pow_suc(w, m)
                w.pow(m.suc) = w * w.pow(m)
                complex_converges_to(complex_pow_seq(z, m.suc), w.pow(m.suc))
                p(m.suc)
            }
        }
        p(Nat.0) and forall(m: Nat) { p(m) implies p(m.suc) }
        alt_induction(p)
        p(k)
    }
}

/// The limit of the k-th powers is the k-th power of the limit.
theorem complex_seq_pow_limit(z: Nat -> Complex, k: Nat) {
    complex_converges(z) implies complex_limit(complex_pow_seq(z, k)) = complex_limit(z).pow(k)
} by {
    if complex_converges(z) {
        complex_converges_imp_converges_to_limit(z)
        complex_converges_to(z, complex_limit(z))
        complex_seq_pow_converges_to(z, complex_limit(z), k)
        complex_converges_to(complex_pow_seq(z, k), complex_limit(z).pow(k))
        complex_converges_to_imp_limit(complex_pow_seq(z, k), complex_limit(z).pow(k))
        complex_limit(complex_pow_seq(z, k)) = complex_limit(z).pow(k)
    }
}

// ---------------------------------------------------------------------------
// d. Eventual boundedness.
// ---------------------------------------------------------------------------

/// The modulus of a difference is bounded by the sum of the moduli:
/// |a| <= |a - b| + |b|.
theorem modulus_le_sub_add(a: Complex, b: Complex) {
    a.modulus <= (a - b).modulus + b.modulus
} by {
    modulus_le_add(a, -b)
    a.modulus <= (a + -b).modulus + (-b).modulus
    a - b = a + -b
    (a + -b).modulus = (a - b).modulus
    modulus_neg(b)
    (-b).modulus = b.modulus
    a.modulus <= (a - b).modulus + b.modulus
}

/// A convergent complex sequence is eventually bounded:
/// eventually |z(n)| <= |w| + 1.
theorem complex_seq_eventually_bounded(z: Nat -> Complex, w: Complex) {
    complex_converges_to(z, w) implies exists(m: Nat) {
        forall(i: Nat) {
            m <= i implies z(i).modulus <= w.modulus + Real.1
        }
    }
} by {
    if complex_converges_to(z, w) {
        complex_converges_to_imp_tendsto_metric(z, w)
        tendsto_metric(z, w)
        real_one_positive
        Real.1 > Real.0
        Real.1.is_positive
        tendsto_metric_tail_bound(z, w, Real.1)
        tendsto_metric(z, w) and Real.1.is_positive implies exists(n: Nat) {
            metric_tail_bound(z, w, n, Real.1)
        }
        exists(n: Nat) {
            metric_tail_bound(z, w, n, Real.1)
        }
        let n: Nat satisfy {
            metric_tail_bound(z, w, n, Real.1)
        }
        forall(i: Nat) {
            if n <= i {
                metric_tail_bound_at(z, w, n, Real.1, i)
                metric_tail_bound(z, w, n, Real.1) and n <= i implies z(i).distance(w) < Real.1
                z(i).distance(w) < Real.1
                complex_distance_eq_modulus_sub(z(i), w)
                z(i).distance(w) = (z(i) - w).modulus
                (z(i) - w).modulus < Real.1
                modulus_reverse_triangle(z(i), w)
                (z(i).modulus - w.modulus).abs <= (z(i) - w).modulus
                lte_lt_trans((z(i).modulus - w.modulus).abs, (z(i) - w).modulus, Real.1)
                (z(i).modulus - w.modulus).abs < Real.1
                lte_abs(z(i).modulus - w.modulus)
                z(i).modulus - w.modulus <= (z(i).modulus - w.modulus).abs
                lte_lt_trans(z(i).modulus - w.modulus, (z(i).modulus - w.modulus).abs, Real.1)
                z(i).modulus - w.modulus < Real.1
                lt_add_right(z(i).modulus - w.modulus, Real.1, w.modulus)
                (z(i).modulus - w.modulus) + w.modulus < Real.1 + w.modulus
                (z(i).modulus - w.modulus) + w.modulus = z(i).modulus
                z(i).modulus < Real.1 + w.modulus
                add_comm(Real.1, w.modulus)
                Real.1 + w.modulus = w.modulus + Real.1
                z(i).modulus < w.modulus + Real.1
                lt_imp_lte(z(i).modulus, w.modulus + Real.1)
                z(i).modulus <= w.modulus + Real.1
            }
        }
        forall(i: Nat) {
            n <= i implies z(i).modulus <= w.modulus + Real.1
        }
        exists(m: Nat) {
            forall(i: Nat) {
                m <= i implies z(i).modulus <= w.modulus + Real.1
            }
        }
    }
}
