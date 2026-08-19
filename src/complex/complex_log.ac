from real import Real, pos_imp_eq_abs
from complex.complex import Complex
from complex.complex_abs import modulus_from_real
from complex.complex_exp import complex_exp, complex_exp_from_real

// ---------------------------------------------------------------------------
// The complex logarithm.
//
// The principal branch of the complex logarithm is z.log = ln|z| + i·arg(z).
// The library has no argument function (grep for "atan"/"arctan" in src/real/
// finds nothing), so the argument term cannot be expressed yet.  The modulus
// contribution ln|z| = (|z|).log.get_or_else(Real.0) is defined below; on the positive real
// axis the argument term vanishes, and there the results below give the
// principal branch and agree with the real logarithm.
//
// The real package's interface (src/real/interface.ac) declares the real
// exponential Real.exp, the real logarithm log and its defaulting value log_or_zero,
// but exports no theorems about them: the proofs in src/real/Real.exp.ac and
// src/real/log.ac are private to the real package.  The theorems below that
// only need the definitions are proved; the inverse laws and special values,
// which additionally need exp_log_or_zero, log_exp, log_some_of_pos_exists, exp_pos
// and exp_injective, are stated at the bottom of this file with their proofs
// commented out.
// ---------------------------------------------------------------------------

/// The complex logarithm of a complex number.  Defined as the real logarithm
/// of the modulus, the modulus contribution of z.log = ln|z| + i·arg(z).
/// On the positive real axis this coincides with the principal branch.
define complex_log(z: Complex) -> Complex {
    Complex.from_real((z.modulus).log.get_or_else(Real.0))
}

attributes Complex {
    /// The complex logarithm.
    let log = complex_log
}

// ---------------------------------------------------------------------------
// Consistency with the real logarithm on the positive real axis.
// ---------------------------------------------------------------------------

/// The complex logarithm of an embedded positive real number is the embedding
/// of the real logarithm: x.log = ln(x) for every positive real x.
theorem complex_log_from_real_pos(x: Real) {
    x > Real.0 implies complex_log(Complex.from_real(x)) = Complex.from_real(x.log.get_or_else(Real.0))
} by {
    if x > Real.0 {
        complex_log(Complex.from_real(x)) = Complex.from_real((Complex.from_real(x).modulus).log.get_or_else(Real.0))
        x.is_positive
        pos_imp_eq_abs(x)
        x = x.abs
        modulus_from_real(x)
        Complex.from_real(x).modulus = x.abs
        Complex.from_real(x).modulus = x
        (Complex.from_real(x).modulus).log.get_or_else(Real.0) = x.log.get_or_else(Real.0)
        Complex.from_real((Complex.from_real(x).modulus).log.get_or_else(Real.0)) = Complex.from_real(x.log.get_or_else(Real.0))
        complex_log(Complex.from_real(x)) = Complex.from_real(x.log.get_or_else(Real.0))
    }
}

// ---------------------------------------------------------------------------
// The inverse properties, in their conditional form.
//
// The complex side of (x.log).exp = x and (x.exp).log = x is proved below,
// with the real-logarithm facts that the real package does not yet export
// taken as hypotheses.  The unconditional statements appear at the bottom of
// this file with their proofs commented out.
// ---------------------------------------------------------------------------

/// The complex exponential of the logarithm of a positive real number is the
/// number itself, whenever the real logarithm is a right inverse of the real
/// exponential: (x.log).exp = x for every positive real x with
/// (x.log.get_or_else(Real.0)).exp = x.
theorem complex_exp_log_pos_cond(x: Real) {
    x > Real.0 and (x.log.get_or_else(Real.0)).exp = x
    implies complex_exp(complex_log(Complex.from_real(x))) = Complex.from_real(x)
} by {
    if x > Real.0 and (x.log.get_or_else(Real.0)).exp = x {
        complex_log_from_real_pos(x)
        x > Real.0 implies complex_log(Complex.from_real(x)) = Complex.from_real(x.log.get_or_else(Real.0))
        complex_log(Complex.from_real(x)) = Complex.from_real(x.log.get_or_else(Real.0))
        complex_exp(complex_log(Complex.from_real(x))) = complex_exp(Complex.from_real(x.log.get_or_else(Real.0)))
        complex_exp_from_real(x.log.get_or_else(Real.0))
        complex_exp(Complex.from_real(x.log.get_or_else(Real.0))) = Complex.from_real((x.log.get_or_else(Real.0)).exp)
        (x.log.get_or_else(Real.0)).exp = x
        Complex.from_real((x.log.get_or_else(Real.0)).exp) = Complex.from_real(x)
        complex_exp(complex_log(Complex.from_real(x))) = Complex.from_real(x)
    }
}

/// The complex logarithm of the complex exponential of an embedded real number
/// is the number itself, whenever the real logarithm recovers exponential
/// values: (x.exp).log = x for every real x with x.exp > 0 and
/// (x.exp).log.get_or_else(Real.0) = x.
theorem complex_log_exp_real_cond(x: Real) {
    x.exp > Real.0 and (x.exp).log.get_or_else(Real.0) = x
    implies complex_log(complex_exp(Complex.from_real(x))) = Complex.from_real(x)
} by {
    if x.exp > Real.0 and (x.exp).log.get_or_else(Real.0) = x {
        complex_exp_from_real(x)
        complex_exp(Complex.from_real(x)) = Complex.from_real(x.exp)
        complex_log(complex_exp(Complex.from_real(x))) = complex_log(Complex.from_real(x.exp))
        complex_log(Complex.from_real(x.exp)) = Complex.from_real((Complex.from_real(x.exp).modulus).log.get_or_else(Real.0))
        x.exp.is_positive
        pos_imp_eq_abs(x.exp)
        x.exp = x.exp.abs
        modulus_from_real(x.exp)
        Complex.from_real(x.exp).modulus = x.exp.abs
        Complex.from_real(x.exp).modulus = x.exp
        (Complex.from_real(x.exp).modulus).log.get_or_else(Real.0) = (x.exp).log.get_or_else(Real.0)
        Complex.from_real((Complex.from_real(x.exp).modulus).log.get_or_else(Real.0)) = Complex.from_real((x.exp).log.get_or_else(Real.0))
        complex_log(Complex.from_real(x.exp)) = Complex.from_real((x.exp).log.get_or_else(Real.0))
        (x.exp).log.get_or_else(Real.0) = x
        Complex.from_real((x.exp).log.get_or_else(Real.0)) = Complex.from_real(x)
        complex_log(complex_exp(Complex.from_real(x))) = Complex.from_real(x)
    }
}

// ---------------------------------------------------------------------------
// The unconditional inverse properties and special values.
//
// These are the intended statements.  They need the following facts about the
// real exponential and logarithm, which are proved in src/real/Real.exp.ac and
// src/real/log.ac but are not exported through the real package interface:
//
//   exp_log_or_zero(x, y) : x > Real.0 and x.log = Option.some(y) implies y.exp = x
//   exp_pos(x)            : x.exp > Real.0
//   log_exp(x)            : (x.exp).log = Option.some(x)
//   log_some_of_pos(x, y) : x > Real.0 and x.log = Option.some(y) implies is_log(x, y)
//   log_some_of_pos_exists(x) : x > Real.0 implies exists(y: Real) { x.log = Option.some(y) }
//   exp_zero              : (Real.0).exp = Real.1
//   exp_injective(x,y)    : x.exp = y.exp implies x = y
//
// Once those theorems are exported, uncomment the statements and proofs below;
// the sketches given should go through unchanged.  (The lemma log_value_exp
// derives (x.exp).log.get_or_else(Real.0) = x from log_exp, log_some_of_pos_exists and
// some_injective.)
// ---------------------------------------------------------------------------

// /// The defaulting logarithm of an exponential value is the argument:
// /// (x.exp).log.get_or_else(Real.0) = x for every real x.
// theorem log_value_exp(x: Real) {
//     (x.exp).log.get_or_else(Real.0) = x
// } by {
//     exp_pos(x)
//     x.exp > Real.0
//     log_some_of_pos_exists(x.exp)
//     let y: Real satisfy {
//         x.exp.log = Option.some(y)
//     }
//     exp_log_or_zero(x.exp, y)
//     y.exp = x.exp
//     log_exp(x)
//     (x.exp).log = Option.some(x)
//     Option.some(y) = Option.some(x)
//     some_injective(y, x)
//     y = x
//     (x.exp).log.get_or_else(Real.0) = x
// }

// /// The complex exponential of the logarithm of a positive real number is the
// /// number itself: (x.log).exp = x for every positive real x.
// theorem complex_exp_log_pos(x: Real) {
//     x > Real.0 implies complex_exp(complex_log(Complex.from_real(x))) = Complex.from_real(x)
// } by {
//     if x > Real.0 {
//         complex_log_from_real_pos(x)
//         x > Real.0 implies complex_log(Complex.from_real(x)) = Complex.from_real(x.log.get_or_else(Real.0))
//         complex_log(Complex.from_real(x)) = Complex.from_real(x.log.get_or_else(Real.0))
//         complex_exp(complex_log(Complex.from_real(x))) = complex_exp(Complex.from_real(x.log.get_or_else(Real.0)))
//         complex_exp_from_real(x.log.get_or_else(Real.0))
//         complex_exp(Complex.from_real(x.log.get_or_else(Real.0))) = Complex.from_real((x.log.get_or_else(Real.0)).exp)
//         log_some_of_pos_exists(x)
//         let y: Real satisfy {
//             x.log = Option.some(y)
//         }
//         exp_log_or_zero(x, y)
//         y.exp = x
//         (x.log.get_or_else(Real.0)).exp = x
//         Complex.from_real((x.log.get_or_else(Real.0)).exp) = Complex.from_real(x)
//         complex_exp(complex_log(Complex.from_real(x))) = Complex.from_real(x)
//     }
// }

// /// The complex logarithm of the complex exponential of an embedded real number
// /// is the number itself: (x.exp).log = x for every real x.  This is the
// /// restricted inverse property, holding on the whole real axis.
// theorem complex_log_exp_real(x: Real) {
//     complex_log(complex_exp(Complex.from_real(x))) = Complex.from_real(x)
// } by {
//     complex_exp_from_real(x)
//     complex_exp(Complex.from_real(x)) = Complex.from_real(x.exp)
//     complex_log(complex_exp(Complex.from_real(x))) = complex_log(Complex.from_real(x.exp))
//     complex_log(Complex.from_real(x.exp)) = Complex.from_real((Complex.from_real(x.exp).modulus).log.get_or_else(Real.0))
//     exp_pos(x)
//     x.exp > Real.0
//     x.exp.is_positive
//     pos_imp_eq_abs(x.exp)
//     x.exp = x.exp.abs
//     modulus_from_real(x.exp)
//     Complex.from_real(x.exp).modulus = x.exp.abs
//     Complex.from_real(x.exp).modulus = x.exp
//     (Complex.from_real(x.exp).modulus).log.get_or_else(Real.0) = (x.exp).log.get_or_else(Real.0)
//     Complex.from_real((Complex.from_real(x.exp).modulus).log.get_or_else(Real.0)) = Complex.from_real((x.exp).log.get_or_else(Real.0))
//     complex_log(Complex.from_real(x.exp)) = Complex.from_real((x.exp).log.get_or_else(Real.0))
//     log_value_exp(x)
//     (x.exp).log.get_or_else(Real.0) = x
//     Complex.from_real((x.exp).log.get_or_else(Real.0)) = Complex.from_real(x)
//     complex_log(complex_exp(Complex.from_real(x))) = Complex.from_real(x)
// }

// /// The complex logarithm of one is zero.
// theorem complex_log_one {
//     complex_log(Complex.1) = Complex.0
// } by {
//     complex_log(Complex.1) = Complex.from_real((Complex.1.modulus).log.get_or_else(Real.0))
//     modulus_of_one
//     Complex.1.modulus = Real.1
//     log_value_exp(Real.0)
//     ((Real.0).exp).log.get_or_else(Real.0) = Real.0
//     exp_zero
//     (Real.0).exp = Real.1
//     (Real.1).log.get_or_else(Real.0) = Real.0
//     (Complex.1.modulus).log.get_or_else(Real.0) = (Real.1).log.get_or_else(Real.0)
//     Complex.from_real((Complex.1.modulus).log.get_or_else(Real.0)) = Complex.from_real((Real.1).log.get_or_else(Real.0))
//     Complex.from_real((Real.1).log.get_or_else(Real.0)) = Complex.from_real(Real.0)
//     Complex.from_real(Real.0) = Complex.0
//     complex_log(Complex.1) = Complex.0
// }

// /// The complex logarithm of the real number e (the exponential of one) is one.
// theorem complex_log_e {
//     complex_log(Complex.from_real((Real.1).exp)) = Complex.1
// } by {
//     complex_log(Complex.from_real((Real.1).exp)) = Complex.from_real((Complex.from_real((Real.1).exp).modulus).log.get_or_else(Real.0))
//     exp_pos(Real.1)
//     (Real.1).exp > Real.0
//     (Real.1).exp.is_positive
//     pos_imp_eq_abs((Real.1).exp)
//     (Real.1).exp = (Real.1).exp.abs
//     modulus_from_real((Real.1).exp)
//     Complex.from_real((Real.1).exp).modulus = (Real.1).exp.abs
//     Complex.from_real((Real.1).exp).modulus = (Real.1).exp
//     (Complex.from_real((Real.1).exp).modulus).log.get_or_else(Real.0) = ((Real.1).exp).log.get_or_else(Real.0)
//     Complex.from_real((Complex.from_real((Real.1).exp).modulus).log.get_or_else(Real.0)) = Complex.from_real(((Real.1).exp).log.get_or_else(Real.0))
//     complex_log(Complex.from_real((Real.1).exp)) = Complex.from_real(((Real.1).exp).log.get_or_else(Real.0))
//     log_value_exp(Real.1)
//     ((Real.1).exp).log.get_or_else(Real.0) = Real.1
//     Complex.from_real(((Real.1).exp).log.get_or_else(Real.0)) = Complex.from_real(Real.1)
//     from_real_one
//     Complex.from_real(Real.1) = Complex.1
//     complex_log(Complex.from_real((Real.1).exp)) = Complex.1
// }
