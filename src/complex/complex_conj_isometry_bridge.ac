from complex.complex import Complex
from complex.complex_abs import modulus_conj
from complex.complex_conj_hom import complex_conj_fn
from complex.complex_conj_linear import complex_conj_fn_sub
from complex.complex_metric import complex_distance_eq_modulus_sub
from analysis import is_isometry

/// Complex conjugation preserves the metric distance on complex numbers.
theorem complex_conj_fn_distance_eq(a: Complex, b: Complex) {
    complex_conj_fn(a).distance(complex_conj_fn(b)) = a.distance(b)
} by {
    complex_distance_eq_modulus_sub(complex_conj_fn(a), complex_conj_fn(b))
    complex_conj_fn(a).distance(complex_conj_fn(b)) =
        (complex_conj_fn(a) - complex_conj_fn(b)).modulus
    complex_conj_fn_sub(a, b)
    complex_conj_fn(a - b) = complex_conj_fn(a) - complex_conj_fn(b)
    (complex_conj_fn(a) - complex_conj_fn(b)).modulus = complex_conj_fn(a - b).modulus
    complex_conj_fn(a - b) = (a - b).conj
    (complex_conj_fn(a) - complex_conj_fn(b)).modulus = (a - b).conj.modulus
    modulus_conj(a - b)
    (a - b).conj.modulus = (a - b).modulus
    (complex_conj_fn(a) - complex_conj_fn(b)).modulus = (a - b).modulus
    complex_distance_eq_modulus_sub(a, b)
    a.distance(b) = (a - b).modulus
    complex_conj_fn(a).distance(complex_conj_fn(b)) = a.distance(b)
}

/// Complex conjugation is an isometry of the complex metric space.
theorem complex_conj_fn_is_isometry {
    is_isometry(complex_conj_fn)
} by {
    is_isometry(complex_conj_fn) = forall(a: Complex, b: Complex) {
        complex_conj_fn(a).distance(complex_conj_fn(b)) = a.distance(b)
    }
    forall(a: Complex, b: Complex) {
        complex_conj_fn_distance_eq(a, b)
    }
}
