from real import Real
from complex.complex import Complex
from complex.complex_abs import modulus_mul
from algebra.module.normed_add_comm_group import NormedAddCommGroup
from algebra.module.normed_field import NormedField

/// The complex modulus gives `Complex` the structure of a normed additive commutative group.
instance Complex: NormedAddCommGroup {
    let norm: Complex -> Real = Complex.modulus
}

/// Multiplication is compatible with the norm on `Complex`.
/// This bridge is included because it is the exact norm-language obligation for
/// the `Complex: NormedField` instance; `modulus_mul` is the underlying theorem.
theorem complex_norm_mul(a: Complex, b: Complex) {
    (a * b).norm = a.norm * b.norm
} by {
    modulus_mul(a, b)
}

/// The complex modulus makes `Complex` a normed field.
instance Complex: NormedField
