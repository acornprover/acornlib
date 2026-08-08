from complex.complex import Complex, is_real_mul, one_is_real
from nat import Nat
from nat import alt_induction
from algebra.ring.ring_hom import ring_hom_pow
from complex.complex_conj_hom import complex_conj_fn, complex_conj_ring_hom, complex_conj_ring_hom_hom

/// Complex conjugation commutes with natural-number powers.
theorem complex_conj_fn_pow(a: Complex, n: Nat) {
    complex_conj_fn(a.pow(n)) = complex_conj_fn(a).pow(n)
} by {
    ring_hom_pow(complex_conj_ring_hom, a, n)
    complex_conj_ring_hom_hom
}

/// Natural-number powers of a real complex number are real.
theorem is_real_pow(a: Complex, n: Nat) {
    a.is_real implies a.pow(n).is_real
} by {
    if a.is_real {
        define g(x: Nat) -> Bool { a.pow(x).is_real }
        a.pow(Nat.0) = Complex.1
        one_is_real
        Complex.1.is_real
        a.pow(Nat.0).is_real
        g(Nat.0)
        forall(x: Nat) {
            if g(x) {
                is_real_mul(a, a.pow(x))
                (a * a.pow(x)).is_real
                g(x.suc)
            }
        }
        alt_induction(g)
        g(n)
        a.pow(n).is_real
    }
}
