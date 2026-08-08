from real import Real
from algebra.add import Add
from algebra.mul import Mul
from algebra.neg import Neg
from algebra.inverse import Inverse

/// Complex numbers consist of a real part and an imaginary part.
/// They extend the real numbers and satisfy the equation i² = -1.
structure Complex {
    /// The real part of the complex number.
    re: Real
    /// The imaginary part of the complex number.
    im: Real
}

/// Adds two complex numbers component-wise.
instance Complex: Add {
    define add(self, other: Complex) -> Complex {
        Complex.new(self.re + other.re, self.im + other.im)
    }
}

/// Multiplies two complex numbers using the formula (a+bi)(c+di) = (ac-bd)+(ad+bc)i.
instance Complex: Mul {
    define mul(self, other: Complex) -> Complex {
        // (a + bi) * (c + di) = (ac - bd) + (ad + bc)i
        Complex.new(
            self.re * other.re - self.im * other.im,
            self.re * other.im + self.im * other.re
        )
    }
}

attributes Complex {
    /// Converts a real number to a complex number (with zero imaginary part).
    let from_real: Real -> Complex = function(r: Real) {
        Complex.new(r, Real.0)
    }

    /// The imaginary unit, satisfying `i² = -1`.
    let i: Complex = Complex.new(Real.0, Real.1)

    /// True if this complex number has no imaginary component.
    define is_real(self) -> Bool {
        self.im = Real.0
    }

    /// True if this complex number is purely imaginary (no real component).
    define is_imaginary(self) -> Bool {
        self.re = Real.0 and self.im != Real.0
    }

    /// Yields the complex conjugate (negates the imaginary part).
    define conj(self) -> Complex {
        Complex.new(self.re, -self.im)
    }

    /// Computes the squared magnitude |z|² = re² + im².
    define abs_squared(self) -> Real {
        self.re * self.re + self.im * self.im
    }
}

from algebra.zero import Zero

instance Complex: Zero {
    let 0: Complex = Complex.new(Real.0, Real.0)
}

from algebra.one import One

instance Complex: One {
    let 1: Complex = Complex.new(Real.1, Real.0)
}

// Alias for convenience
let i = Complex.i

// Theorems for complex numbers

// Addition properties
theorem add_comm(a: Complex, b: Complex) {
    a + b = b + a
}

theorem add_assoc(a: Complex, b: Complex, c: Complex) {
    (a + b) + c = a + (b + c)
} by {
    // Component-wise expansion
    let left_re1 = (a + b).re + c.re
    let left_im1 = (a + b).im + c.im
    let left_re2 = (a.re + b.re) + c.re
    let left_im2 = (a.im + b.im) + c.im

    let right_re1 = a.re + (b + c).re
    let right_im1 = a.im + (b + c).im
    let right_re2 = a.re + (b.re + c.re)
    let right_im2 = a.im + (b.im + c.im)

    // Apply real number associativity
      // Real.assoc_add

      // Real.assoc_add

    // Build final complexes
    let final_left = Complex.new(left_re2, left_im2)
    let final_right = Complex.new(right_re2, right_im2)

    // Component-wise equality
    a.im + (b.im + c.im) = a.im + b.im + c.im
    a.re + (b.re + c.re) = a.re + b.re + c.re
    Complex.new(a.re + b.re, a.im + b.im).im = a.im + b.im
    Complex.new(b.re + c.re, b.im + c.im).im = b.im + c.im
    Complex.new(a.re + b.re, a.im + b.im).re = a.re + b.re
    Complex.new(b.re + c.re, b.im + c.im).re = b.re + c.re
}

theorem add_zero_right(a: Complex) {
    a + Complex.0 = a
} by {
    Complex.new(Real.0, Real.0).im = Real.0
    Complex.new(Real.0, Real.0).re = Real.0
}

theorem add_zero_left(a: Complex) {
    Complex.0 + a = a
}

// Multiplication properties
theorem mul_comm(a: Complex, b: Complex) {
    a * b = b * a
}

// Distributive property of multiplication over addition
theorem distrib(a: Complex, b: Complex, c: Complex) {
    a * (b + c) = a * b + a * c
} by {
    let left = a * (b + c)
    let right = a * b + a * c

    // Expand (b + c)
    (b + c).re = b.re + c.re
    (b + c).im = b.im + c.im

    // Simplify left.re
    left.re = a.re * (b + c).re - a.im * (b + c).im
    a.re * (b.re + c.re) = a.re * b.re + a.re * c.re
    a.im * (b.im + c.im) = a.im * b.im + a.im * c.im
    left.re = a.re * b.re + a.re * c.re - a.im * b.im - a.im * c.im

    // Simplify left.im
    left.im = a.re * (b + c).im + a.im * (b + c).re
    a.re * (b.im + c.im) = a.re * b.im + a.re * c.im
    a.im * (b.re + c.re) = a.im * b.re + a.im * c.re
    left.im = a.re * b.im + a.re * c.im + a.im * b.re + a.im * c.re

    // Simplify right.re
    (a * b).re = a.re * b.re - a.im * b.im
    (a * c).re = a.re * c.re - a.im * c.im
    right.re = (a * b).re + (a * c).re
    right.re = a.re * b.re - a.im * b.im + a.re * c.re - a.im * c.im
    a.re * b.re - a.im * b.im + a.re * c.re - a.im * c.im = a.re * b.re + a.re * c.re - a.im * b.im - a.im * c.im

    // Simplify right.im
    (a * b).im = a.re * b.im + a.im * b.re
    (a * c).im = a.re * c.im + a.im * c.re
    right.im = (a * b).im + (a * c).im
    right.im = a.re * b.im + a.im * b.re + a.re * c.im + a.im * c.re
    // Rearrange terms
    a.im * b.re + a.re * c.im = a.re * c.im + a.im * b.re
    a.re * b.im + a.im * b.re + a.re * c.im + a.im * c.re = a.re * b.im + a.re * c.im + a.im * b.re + a.im * c.re
}

theorem mul_three_re(a: Complex, b: Complex, c: Complex) {
    ((a * b) * c).re = a.re * b.re * c.re - a.re * b.im * c.im - a.im * b.re * c.im - a.im * b.im * c.re
} by {
    (a * b).re = a.re * b.re - a.im * b.im
    (a * b).im = a.re * b.im + a.im * b.re
    ((a * b) * c).re = (a * b).re * c.re - (a * b).im * c.im
    ((a * b) * c).re = (a.re * b.re - a.im * b.im) * c.re - (a.re * b.im + a.im * b.re) * c.im
    (a.re * b.re - a.im * b.im) * c.re = a.re * b.re * c.re - a.im * b.im * c.re
    (a.re * b.im + a.im * b.re) * c.im = a.re * b.im * c.im + a.im * b.re * c.im
}

theorem mul_three_im(a: Complex, b: Complex, c: Complex) {
    ((a * b) * c).im = a.re * b.re * c.im + a.re * b.im * c.re + a.im * b.re * c.re - a.im * b.im * c.im
} by {
    (a * b).re = a.re * b.re - a.im * b.im
    (a * b).im = a.re * b.im + a.im * b.re
    ((a * b) * c).im = (a * b).re * c.im + (a * b).im * c.re
    ((a * b) * c).im = (a.re * b.re - a.im * b.im) * c.im + (a.re * b.im + a.im * b.re) * c.re
    (a.re * b.re - a.im * b.im) * c.im = a.re * b.re * c.im - a.im * b.im * c.im
    (a.re * b.im + a.im * b.re) * c.re = a.re * b.im * c.re + a.im * b.re * c.re
}

theorem mul_assoc(a: Complex, b: Complex, c: Complex) {
    (a * b) * c = a * (b * c)
} by {
    // The real parts are equal
    a.im * (b.im * c.re) = a.im * b.im * c.re
    a.im * (b.re * c.im) = a.im * b.re * c.im
    a.re * (b.im * c.im) = a.re * b.im * c.im
    a.re * (b.re * c.re) = a.re * b.re * c.re
    a.re * b.re * c.re - a.im * b.re * c.im - a.re * b.im * c.im = a.re * b.re * c.re - a.re * b.im * c.im - a.im * b.re * c.im
    b.re * c.re * a.re - b.re * c.im * a.im - b.im * c.im * a.re - b.im * c.re * a.im = b.re * c.re * a.re - b.re * c.im * a.im - b.im * c.re * a.im - b.im * c.im * a.re
    a * (b * c) = b * c * a
    a.im * (b.im * c.re) = b.im * c.re * a.im
    a.im * (b.re * c.im) = b.re * c.im * a.im
    a.re * (b.im * c.im) = b.im * c.im * a.re
    a.re * (b.re * c.re) = b.re * c.re * a.re
    a.re * b.re * c.re - a.re * b.im * c.im - a.im * b.re * c.im - a.im * b.im * c.re = (a * b * c).re
    b.re * c.re * a.re - b.re * c.im * a.im - b.im * c.re * a.im - b.im * c.im * a.re = (b * c * a).re
    ((a * b) * c).re = (a * (b * c)).re

    // The imaginary parts are equal
    a.im * (b.im * c.im) = a.im * b.im * c.im
    a.im * (b.re * c.re) = a.im * b.re * c.re
    a.re * (b.im * c.re) = a.re * b.im * c.re
    a.re * (b.re * c.im) = a.re * b.re * c.im
    b.re * c.re * a.im + (b.re * c.im * a.re + b.im * c.re * a.re) = b.re * c.re * a.im + b.re * c.im * a.re + b.im * c.re * a.re
    a.im * (b.im * c.im) = b.im * c.im * a.im
    a.im * (b.re * c.re) = b.re * c.re * a.im
    a.re * (b.im * c.re) = b.im * c.re * a.re
    a.re * (b.re * c.im) = b.re * c.im * a.re
    a.im * b.re * c.re + (a.re * b.re * c.im + a.re * b.im * c.re) = a.re * b.re * c.im + a.re * b.im * c.re + a.im * b.re * c.re
    a.re * b.re * c.im + a.re * b.im * c.re + a.im * b.re * c.re - a.im * b.im * c.im = (a * b * c).im
    b.re * c.re * a.im + b.re * c.im * a.re + b.im * c.re * a.re - b.im * c.im * a.im = (b * c * a).im
    ((a * b) * c).im = (a * (b * c)).im
}

/// Yields the additive inverse of this complex number.
instance Complex: Neg {
    define neg(self) -> Complex {
        Complex.new(-self.re, -self.im)
    }
}

theorem neg_one {
    -Complex.1 = Complex.new(-Real.1, Real.0)
} by {
    Complex.1.re = Real.1
    Complex.1.im = Real.0
}

theorem neg_re(a: Complex) {
    -(a.re) = (-a).re
}

theorem neg_im(a: Complex) {
    -(a.im) = (-a).im
}

theorem neg_one_lifts {
    -Complex.1 = Complex.from_real(-Real.1)
}

// Properties of i
theorem i_squared {
    Complex.i * Complex.i = Complex.new(-Real.1, Real.0)
} by {
    Complex.new(Real.0, Real.1).re = Real.0
    Complex.new(Real.0, Real.1).im = Real.1
    Complex.new(Real.0 * Real.0 - Real.1 * Real.1, Real.0 * Real.1 + Real.1 * Real.0) = Complex.new(-Real.1, Real.0)
}

theorem conj_mul(a: Complex, b: Complex) {
    (a * b).conj = a.conj * b.conj
} by {
    a.conj = Complex.new(a.re, -a.im)
    b.conj = Complex.new(b.re, -b.im)
    Complex.new(a.re, -a.im).re = a.re
    Complex.new(a.re, -a.im).im = -a.im
    Complex.new(b.re, -b.im).re = b.re
    Complex.new(b.re, -b.im).im = -b.im

    // Calculate components of a.conj * b.conj
    let real_part: Real = a.re * b.re - (-a.im) * (-b.im)
    let imag_part: Real = a.re * (-b.im) + (-a.im) * b.re

    // Simplify real_part
    -a.im * -b.im = a.im * b.im
    (a * b).re = a.re * b.re - a.im * b.im

    // Simplify imag_part
    -(a.re * b.im) + -(a.im * b.re) = -(a.re * b.im + a.im * b.re)
    imag_part = -((a * b).im)

}

theorem conj_conj(a: Complex) {
    a.conj.conj = a
} by {
    Complex.new(a.re, -a.im).re = a.re
    Complex.new(a.re, -a.im).im = -a.im
}

// Absolute value properties
theorem abs_squared_conj(a: Complex) {
    a * a.conj = Complex.new(a.abs_squared, Real.0)
} by {
    Complex.new(a.re, -a.im).re = a.re
    Complex.new(a.re, -a.im).im = -a.im
    a.im * -a.im = -(a.im * a.im)
    a.re * a.re - -(a.im * a.im) = a.re * a.re + a.im * a.im
    a.re * -a.im = -(a.re * a.im)
    a.im * a.re = a.re * a.im
    -(a.re * a.im) + a.re * a.im = Real.0
}

// Absolute value squared is non-negative
theorem abs_squared_nonneg(a: Complex) {
    a.abs_squared >= Real.0
} by {
    // Definition of abs_squared

    // Define variables for clearer proof steps
    let re_square: Real = a.re * a.re
    let im_square: Real = a.im * a.im

    // Squares of real numbers are non-negative

    // By definition of re_square and im_square

    // Therefore
    re_square >= Real.0
    im_square >= Real.0

    // If x >= 0 and y >= 0, then x + y >= 0
    re_square + im_square >= Real.0

    // By definition of abs_squared
}

// Multiplicative identity
theorem mul_one_right(a: Complex) {
    a * Complex.1 = a
} by {
    Complex.new(Real.1, Real.0).re = Real.1
    Complex.new(Real.1, Real.0).im = Real.0
    a.im * Real.0 = Real.0
    a.re * Real.0 = Real.0
    a.re - Real.0 = a.re
}

theorem mul_one_left(a: Complex) {
    Complex.1 * a = a
}

// Conjugate distribution over addition
theorem conj_add(a: Complex, b: Complex) {
    (a + b).conj = a.conj + b.conj
} by {
    a + b = Complex.new(a.re + b.re, a.im + b.im)
    (a + b).conj = Complex.new(a.re + b.re, -(a.im + b.im))
    -(a.im + b.im) = -a.im + -b.im
    a.conj.re = a.re
    b.conj.re = b.re
    a.conj.im = -a.im
    b.conj.im = -b.im
}

theorem conj_from_real(a: Real) {
    Complex.from_real(a).conj = Complex.from_real(a)
} by {
    Complex.from_real(a).im = Real.0
    -Real.0 = Real.0
}

theorem abs_squared_from_real(a: Real) {
    Complex.from_real(a).abs_squared = a * a
} by {
    Complex.from_real(a).re = a
    Complex.from_real(a).im = Real.0
    Real.0 * Real.0 = Real.0
}

/// Computes the multiplicative inverse (1/z) for non-zero complex numbers.
/// Yields 0 when applied to 0 (division by zero yields zero).
instance Complex: Inverse {
    define inverse(self) -> Complex {
        self.conj * Complex.from_real(self.abs_squared.inverse)
    }
}

theorem zero_inverse {
    Complex.0.inverse = Complex.0
} by {
    Complex.0.re = Real.0
    Complex.0.im = Real.0
    -Real.0 = Real.0
    Complex.0.conj = Complex.0
    Real.0 * Real.0 = Real.0
    Complex.from_real(Complex.0.abs_squared.inverse) = Complex.from_real(Real.0)
    Complex.from_real(Real.0) = Complex.0
    Real.0 - Real.0 = Real.0
}

theorem mul_from_real(c: Complex, r: Real) {
    c * Complex.from_real(r) = Complex.new(c.re * r, c.im * r)
} by {
    Real.0 * c.im = Real.0
    Real.0 * c.re = Real.0
    c * Complex.new(r, Real.0) = Complex.new(c.re * r - Real.0, c.im * r + Real.0)
    c.re * r - Real.0 = c.re * r
    c.im * r + Real.0 = c.im * r
}

theorem real_mul_lifts(a: Real, b: Real) {
    Complex.from_real(a * b) = Complex.from_real(a) * Complex.from_real(b)
} by {
    Complex.new(a, Real.0).re = a
    Complex.new(a, Real.0).im = Real.0
    Real.0 * b = Real.0
}

theorem real_add_lifts(a: Real, b: Real) {
    Complex.from_real(a + b) = Complex.from_real(a) + Complex.from_real(b)
} by {
    Complex.from_real(a).re = a
    Complex.from_real(b).re = b
    Complex.from_real(a).im = Real.0
    Complex.from_real(b).im = Real.0
}

theorem neg_eq_mul_neg_one(a: Complex) {
    -a = (-Complex.1) * a
}

theorem abs_sq_zero_imp_zero(a: Complex) {
    a.abs_squared = Real.0 implies a = Complex.0
} by {
    a.abs_squared = a.re * a.re + a.im * a.im
    a.re * a.re + a.im * a.im = Real.0
    Real.0 <= a.re * a.re
    a.im * a.im = Real.0 - a.re * a.re
    Real.0 - a.re * a.re <= Real.0
    a.im * a.im <= Real.0
    a.im = Real.0
    a.re * a.re = Real.0
    a.re = Real.0
}

theorem mul_inverse(a: Complex) {
    a != Complex.0 implies a * a.inverse = Complex.1
} by {
    Complex.from_real(a.abs_squared) * Complex.from_real(a.abs_squared.inverse) = Complex.from_real(a.abs_squared * a.abs_squared.inverse)
    Complex.new(a.abs_squared, Real.0) = a * a.conj
    a.abs_squared * a.abs_squared.inverse = Real.1 or a.abs_squared = Real.0
    a.abs_squared != Real.0 or Complex.0 = a
    Complex.new(Complex.1.re, Real.0) = Complex.from_real(Complex.1.re)
    Complex.new(a.abs_squared, Real.0) = Complex.from_real(a.abs_squared)
    Complex.new(Real.1, Real.0).re = Real.1
}

theorem mul_right_cancel(a: Complex, b: Complex, c: Complex) {
    a * c = b * c and c != Complex.0 implies a = b
} by {
    a * c * c.inverse = a * (c * c.inverse)
    b * c * c.inverse = b * (c * c.inverse)
    c * c.inverse = Complex.1 or Complex.0 = c
    a * Complex.1 = Complex.1 * a
    b * Complex.1 = Complex.1 * b
    Complex.1 * Complex.new(a.re, a.im) = Complex.new(a.re, a.im)
    Complex.1 * Complex.new(b.re, b.im) = Complex.new(b.re, b.im)
}

theorem mul_left_cancel(a: Complex, b: Complex, c: Complex) {
    a * b = a * c and a != Complex.0 implies b = c
}

theorem inverse_lifts(a: Real) {
    Complex.from_real(a).inverse = Complex.from_real(a.inverse)
} by {
    // Key definitions that help the prover
    Complex.new(a, Real.0).re = a
    Complex.new(a, Real.0).im = Real.0
    Complex.from_real(a) = Complex.new(a, Real.0)
    Complex.new(Real.0.inverse, Real.0) = Complex.from_real(Real.0.inverse)
    Real.1 + Real.0 = Real.1
    Real.1 * Real.0 = Real.0
    Complex.from_real(a).re = a
    Complex.from_real(a).im = Real.0
    Complex.from_real(a).abs_squared = a * a
    Complex.from_real(a).conj * Complex.from_real(Complex.from_real(a).abs_squared.inverse) = Complex.from_real(a).inverse
    Complex.from_real(a).conj = Complex.from_real(a)

    // Compute the mul_from_real result
    Complex.new(Complex.from_real(a).conj.re * Complex.from_real(a).abs_squared.inverse, Complex.from_real(a).conj.im * Complex.from_real(a).abs_squared.inverse) = Complex.from_real(a).conj * Complex.from_real(Complex.from_real(a).abs_squared.inverse)

    if a = Real.0 {
        Complex.from_real(a).inverse = Complex.0
    } else {
        a != Real.0
        Complex.0 = Complex.new(Real.0, Real.0)
        Complex.new(a, Real.0) != Complex.new(Real.0, Real.0)
        Complex.from_real(a) != Complex.0

        // Compute the inverse of Complex.from_real(a)
        Complex.from_real(a).inverse = Complex.from_real(a) * Complex.from_real((a * a).inverse)

        // Use real_mul_lifts
        Complex.from_real(a) * Complex.from_real((a * a).inverse) = Complex.from_real(a * (a * a).inverse)

        // Show a * (a * a).inverse = a.inverse
        (a * a).inverse * (a * a) = Real.1 or a * a = Real.0
        a * a != Real.0
        (a * a).inverse * (a * a) = Real.1
        a * (a * a).inverse * a = Real.1
        a.inverse * a = Real.1
        a * (a * a).inverse = a.inverse

        Complex.from_real(a).inverse = Complex.from_real(a.inverse)
    }
}

attributes Complex {
    /// Divides this complex number by another.
    /// Division by zero returns zero (making this a total function).
    define div(self, other: Complex) -> Complex {
        self * other.inverse
    }
}

// Demonstrating that Complex is a field.

from algebra.add_semigroup import AddSemigroup

instance Complex: AddSemigroup

from algebra.add_comm_semigroup import AddCommSemigroup

instance Complex: AddCommSemigroup

from algebra.add_monoid import AddMonoid

instance Complex: AddMonoid

from algebra.add_comm_monoid import AddCommMonoid

instance Complex: AddCommMonoid

from algebra.semigroup import Semigroup

instance Complex: Semigroup

from algebra.monoid.monoid import Monoid

instance Complex: Monoid

theorem mul_zero_right(a: Complex) {
    a * Complex.0 = Complex.0
} by {
    a * Complex.new(Real.0, Real.0) = Complex.new(Real.0, Real.0)
}

theorem mul_zero_left(a: Complex) {
    Complex.0 * a = Complex.0
}

from semiring import Semiring

theorem left_distrib(a: Complex, b: Complex, c: Complex) {
    (a + b) * c = a * c + b * c
}

instance Complex: Semiring

from algebra.add_group import AddGroup

theorem add_neg_cancel(a: Complex) {
    a + -a = Complex.0
} by {
    a.re + -a.re = Real.0
    a.im + -a.im = Real.0
}

instance Complex: AddGroup

from algebra.add_comm_group import AddCommGroup

instance Complex: AddCommGroup

from algebra.ring.ring import Ring

instance Complex: Ring

from algebra.comm_semigroup import CommSemigroup

instance Complex: CommSemigroup

from algebra.comm_monoid import CommMonoid

instance Complex: CommMonoid

from comm_ring import CommRing

instance Complex: CommRing

from algebra.field.field import Field, unique_inverse

theorem zero_is_different_than_one {
    Complex.0 != Complex.1
} by {
    Complex.new(Real.0, Real.0).re = Real.0
    Complex.new(Real.1, Real.0).re = Real.1
}

instance Complex: Field

// Component lemmas

theorem eq_by_components(a: Complex, b: Complex) {
    a.re = b.re and a.im = b.im implies a = b
} by {
}

theorem re_add(a: Complex, b: Complex) {
    (a + b).re = a.re + b.re
}

theorem im_add(a: Complex, b: Complex) {
    (a + b).im = a.im + b.im
}

theorem re_neg(a: Complex) {
    (-a).re = -a.re
}

theorem im_neg(a: Complex) {
    (-a).im = -a.im
}

// Conjugate basics

theorem conj_zero {
    Complex.0.conj = Complex.0
} by {
    Complex.0.im = Real.0
    -Real.0 = Real.0
}

theorem conj_one {
    Complex.1.conj = Complex.1
} by {
    Complex.1.im = Real.0
    -Real.0 = Real.0
}

theorem conj_neg(a: Complex) {
    (-a).conj = -(a.conj)
} by {
    a.conj.re = a.re
    a.conj.im = -a.im
}

// is_real characterization

theorem is_real_imp_conj_eq(a: Complex) {
    a.is_real implies a.conj = a
} by {
    a.im = Real.0
    a.conj = Complex.new(a.re, Real.0)
}

/// A complex number whose conjugate equals itself is real.
theorem conj_eq_imp_is_real(a: Complex) {
    a.conj = a implies a.is_real
} by {
    a.conj = Complex.new(a.re, -a.im)
    a.conj.im = -a.im
    if a.conj = a {
        a.im = -a.im
        // Build up a.re * a.re + a.im * a.im = Real.0 by abusing abs_sq_zero_imp_zero shape
        // Use: a.im = -a.im implies a.im * a.im = 0
        let s: Real = a.im * a.im
        s = a.im * -a.im
        a.im * -a.im = -(a.im * a.im)
        s = -s
        s + s = Real.0
        Real.0 <= s
        s = Real.0 - s
        Real.0 - s <= Real.0
        s <= Real.0
        s = Real.0
        a.im = Real.0
    }
}

/// The complex zero is real.
theorem zero_is_real {
    Complex.0.is_real
} by {
}

/// The complex one is real.
theorem one_is_real {
    Complex.1.is_real
} by {
}

/// The sum of two real complex numbers is real.
theorem is_real_add(a: Complex, b: Complex) {
    a.is_real and b.is_real implies (a + b).is_real
} by {
    if a.is_real and b.is_real {
        a.im = Real.0
        b.im = Real.0
        (a + b).im = Real.0
    }
}

/// The negation of a real complex number is real.
theorem is_real_neg(a: Complex) {
    a.is_real implies (-a).is_real
} by {
    if a.is_real {
        -a.im = -Real.0
        (-a).im = Real.0
    }
}

/// The product of two real complex numbers is real.
theorem is_real_mul(a: Complex, b: Complex) {
    a.is_real and b.is_real implies (a * b).is_real
} by {
    if a.is_real and b.is_real {
        a.re * b.im = a.re * Real.0
        a.im * b.re = Real.0 * b.re
        (a * b).im = Real.0
    }
}

/// The conjugate of a real complex number is real.
theorem is_real_conj(a: Complex) {
    a.is_real implies a.conj.is_real
} by {
    if a.is_real {
        -a.im = -Real.0
        a.conj.im = Real.0
    }
}

/// Real-part embedding produces real complex numbers.
theorem from_real_is_real(r: Real) {
    Complex.from_real(r).is_real
} by {
}

// abs_squared identities

theorem abs_squared_zero {
    Complex.0.abs_squared = Real.0
} by {
    Real.0 * Real.0 = Real.0
}

theorem abs_squared_one {
    Complex.1.abs_squared = Real.1
} by {
    Complex.1.re = Real.1
    Complex.1.im = Real.0
    Real.0 * Real.0 = Real.0
}

theorem abs_squared_i {
    Complex.i.abs_squared = Real.1
} by {
    Complex.i.re = Real.0
    Complex.i.im = Real.1
    Real.0 * Real.0 = Real.0
}

theorem abs_squared_neg(a: Complex) {
    (-a).abs_squared = a.abs_squared
} by {
    -a.re * -a.re = -(a.re * -a.re)
    a.re * -a.re = -(a.re * a.re)
    -(-(a.re * a.re)) = a.re * a.re
    -a.im * -a.im = -(a.im * -a.im)
    a.im * -a.im = -(a.im * a.im)
    -(-(a.im * a.im)) = a.im * a.im
}

theorem abs_squared_conj_eq(a: Complex) {
    a.conj.abs_squared = a.abs_squared
} by {
    a.conj.re = a.re
    a.conj.im = -a.im
    -a.im * -a.im = -(a.im * -a.im)
    a.im * -a.im = -(a.im * a.im)
    -(-(a.im * a.im)) = a.im * a.im
}

// i identities

theorem i_squared_eq_neg_one {
    Complex.i * Complex.i = -Complex.1
} by {
    -Complex.1 = Complex.new(-Real.1, Real.0)
}

theorem mul_i(a: Complex) {
    Complex.i * a = Complex.new(-a.im, a.re)
} by {
    Complex.i.re = Real.0
    Complex.i.im = Real.1
    Real.0 * a.re = Real.0
    Real.1 * a.im = a.im
    Real.0 - a.im = -a.im
    Real.0 * a.im = Real.0
    Real.1 * a.re = a.re
    Real.0 + a.re = a.re
}

/// The real part of a difference is the difference of real parts.
theorem re_sub(a: Complex, b: Complex) {
    (a - b).re = a.re - b.re
} by {
    a - b = a + -b
    (-b).re = -b.re
    (a + -b).re = a.re + (-b).re
    a.re + -b.re = a.re - b.re
}

/// The imaginary part of a difference is the difference of imaginary parts.
theorem im_sub(a: Complex, b: Complex) {
    (a - b).im = a.im - b.im
} by {
    a - b = a + -b
    (-b).im = -b.im
    (a + -b).im = a.im + (-b).im
    a.im + -b.im = a.im - b.im
}

/// The difference of two real complex numbers is real.
theorem is_real_sub(a: Complex, b: Complex) {
    a.is_real and b.is_real implies (a - b).is_real
} by {
    if a.is_real and b.is_real {
        a.im = Real.0
        b.im = Real.0
        im_sub(a, b)
        Real.0 - Real.0 = Real.0
        (a - b).im = Real.0
    }
}

/// Conjugation preserves differences.
theorem conj_sub(a: Complex, b: Complex) {
    (a - b).conj = a.conj - b.conj
} by {
    a - b = a + -b
    (a + -b).conj = a.conj + (-b).conj
    a.conj + -(b.conj) = a.conj - b.conj
}

/// Division by one fixes a complex number.
theorem div_one(a: Complex) {
    a / Complex.1 = a
} by {
    Complex.1 * Complex.1.inverse = Complex.1
}

/// A nonzero complex number divided by itself is one.
theorem div_self(a: Complex) {
    a != Complex.0 implies a / a = Complex.1
} by {
}

/// The embedding of real numbers into complex numbers preserves division.
theorem div_from_real(a: Real, b: Real) {
    Complex.from_real(a) / Complex.from_real(b) = Complex.from_real(a / b)
} by {
    Complex.from_real(a) * Complex.from_real(b.inverse) = Complex.from_real(a * b.inverse)
}

/// The real part of the inverse of a complex number.
theorem re_inverse(a: Complex) {
    a.inverse.re = a.re * a.abs_squared.inverse
} by {
    a.conj.re = a.re
    a.conj.im = -a.im
    a.conj * Complex.from_real(a.abs_squared.inverse) =
        Complex.new(a.conj.re * a.abs_squared.inverse, a.conj.im * a.abs_squared.inverse)
    a.inverse = Complex.new(a.re * a.abs_squared.inverse, (-a.im) * a.abs_squared.inverse)
}

/// The imaginary part of the inverse of a complex number.
theorem im_inverse(a: Complex) {
    a.inverse.im = -a.im * a.abs_squared.inverse
} by {
    a.conj.re = a.re
    a.conj.im = -a.im
    a.conj * Complex.from_real(a.abs_squared.inverse) =
        Complex.new(a.conj.re * a.abs_squared.inverse, a.conj.im * a.abs_squared.inverse)
    a.inverse = Complex.new(a.re * a.abs_squared.inverse, (-a.im) * a.abs_squared.inverse)
    (-a.im) * a.abs_squared.inverse = -a.im * a.abs_squared.inverse
}

/// The real part of a complex quotient.
theorem re_div(a: Complex, b: Complex) {
    (a / b).re = (a.re * b.re + a.im * b.im) * b.abs_squared.inverse
} by {
    a / b = a * b.inverse
    (a / b).re = a.re * b.inverse.re - a.im * b.inverse.im
    b.inverse.re = b.re * b.abs_squared.inverse
    b.inverse.im = -b.im * b.abs_squared.inverse
    (a / b).re =
        a.re * (b.re * b.abs_squared.inverse) -
        a.im * (-b.im * b.abs_squared.inverse)
    a.re * (b.re * b.abs_squared.inverse) = a.re * b.re * b.abs_squared.inverse
    -b.im * b.abs_squared.inverse = -(b.im * b.abs_squared.inverse)
    a.im * (-(b.im * b.abs_squared.inverse)) = -(a.im * (b.im * b.abs_squared.inverse))
    a.im * (b.im * b.abs_squared.inverse) = a.im * b.im * b.abs_squared.inverse
    a.im * (-b.im * b.abs_squared.inverse) = -(a.im * b.im * b.abs_squared.inverse)
    a.re * b.re * b.abs_squared.inverse - -(a.im * b.im * b.abs_squared.inverse) =
        a.re * b.re * b.abs_squared.inverse + a.im * b.im * b.abs_squared.inverse
}

/// The imaginary part of a complex quotient.
theorem im_div(a: Complex, b: Complex) {
    (a / b).im = (a.im * b.re - a.re * b.im) * b.abs_squared.inverse
} by {
    (a / b).im = a.re * b.inverse.im + a.im * b.inverse.re
    a.re * (-(b.im * b.abs_squared.inverse)) = -(a.re * (b.im * b.abs_squared.inverse))
    -(a.re * b.im * b.abs_squared.inverse) + a.im * b.re * b.abs_squared.inverse =
        a.im * b.re * b.abs_squared.inverse - a.re * b.im * b.abs_squared.inverse
}

/// The inverse of a real complex number is real.
theorem is_real_inverse(a: Complex) {
    a.is_real implies a.inverse.is_real
} by {
    if a.is_real {
        -a.im = -Real.0
        a.inverse.im = Real.0 * a.abs_squared.inverse
        a.inverse.im = Real.0
    }
}

/// The quotient of two real complex numbers is real.
theorem is_real_div(a: Complex, b: Complex) {
    a.is_real and b.is_real implies (a / b).is_real
} by {
    if a.is_real and b.is_real {
        is_real_inverse(b)
        is_real_mul(a, b.inverse)
        (a / b).is_real
    }
}

/// Division of embedded real numbers produces a real complex number.
theorem from_real_div_is_real(a: Real, b: Real) {
    (Complex.from_real(a) / Complex.from_real(b)).is_real
} by {
    from_real_is_real(a)
    from_real_is_real(b)
    is_real_div(Complex.from_real(a), Complex.from_real(b))
}

/// The squared absolute value of a self-difference is zero.
theorem abs_squared_sub_self(a: Complex) {
    (a - a).abs_squared = Real.0
} by {
    a - a = Complex.0
}

/// The real part of zero is zero.
theorem re_zero {
    Complex.0.re = Real.0
}

/// Multiplication of real-only complex numbers produces a real-only product.
theorem from_real_mul_from_real(x: Real, y: Real) {
    Complex.new(x, Real.0) * Complex.new(y, Real.0) = Complex.new(x * y, Real.0)
} by {
    Real.0 * Real.0 = Real.0
    x * Real.0 = Real.0
    Real.0 * y = Real.0
    x * y - Real.0 = x * y
}

/// The squared absolute value is multiplicative.
theorem abs_squared_mul(a: Complex, b: Complex) {
    (a * b).abs_squared = a.abs_squared * b.abs_squared
} by {
    (a * b) * (a.conj * b.conj) = a * (b * (a.conj * b.conj))
    (a.conj * b) * b.conj = a.conj * (b * b.conj)
    a * (a.conj * (b * b.conj)) = (a * a.conj) * (b * b.conj)
    Complex.new((a * b).abs_squared, Real.0) = Complex.new(a.abs_squared * b.abs_squared, Real.0)
    Complex.new((a * b).abs_squared, Real.0).re = (a * b).abs_squared
    Complex.new(a.abs_squared * b.abs_squared, Real.0).re = a.abs_squared * b.abs_squared
}

/// The squared absolute value of an inverse.
theorem abs_squared_inverse(a: Complex) {
    a.inverse.abs_squared = a.abs_squared.inverse
} by {
    if a = Complex.0 {
        a.abs_squared.inverse = Real.0
    }
    if a != Complex.0 {
        (a * a.inverse).abs_squared = Complex.1.abs_squared
        a.abs_squared * a.inverse.abs_squared = Real.1
        unique_inverse(a.abs_squared, a.inverse.abs_squared)
        a.inverse.abs_squared = a.abs_squared.inverse
    }
}

/// The squared absolute value of a quotient.
theorem abs_squared_div(a: Complex, b: Complex) {
    (a / b).abs_squared = a.abs_squared / b.abs_squared
} by {
    (a / b).abs_squared = a.abs_squared * b.inverse.abs_squared
}

/// The imaginary part of zero is zero.
theorem im_zero {
    Complex.0.im = Real.0
}

/// The real part of one is one.
theorem re_one {
    Complex.1.re = Real.1
}

/// The imaginary part of one is zero.
theorem im_one {
    Complex.1.im = Real.0
}

/// The real part of i is zero.
theorem re_i {
    Complex.i.re = Real.0
}

/// The imaginary part of i is one.
theorem im_i {
    Complex.i.im = Real.1
}

/// The real part of an embedded real number is itself.
theorem re_from_real(a: Real) {
    Complex.from_real(a).re = a
}

/// The imaginary part of an embedded real number is zero.
theorem im_from_real(a: Real) {
    Complex.from_real(a).im = Real.0
}

/// The real part of a product follows the (ac - bd) rule.
theorem re_mul(a: Complex, b: Complex) {
    (a * b).re = a.re * b.re - a.im * b.im
}

/// The imaginary part of a product follows the (ad + bc) rule.
theorem im_mul(a: Complex, b: Complex) {
    (a * b).im = a.re * b.im + a.im * b.re
}

/// Conjugation preserves the real part.
theorem re_conj(a: Complex) {
    a.conj.re = a.re
}

/// Conjugation negates the imaginary part.
theorem im_conj(a: Complex) {
    a.conj.im = -a.im
}

/// Squared absolute value equals re² + im².
theorem abs_squared_eq(a: Complex) {
    a.abs_squared = a.re * a.re + a.im * a.im
}

/// The squared absolute value is zero exactly when the complex number is zero.
theorem abs_squared_eq_zero(a: Complex) {
    a.abs_squared = Real.0 iff a = Complex.0
} by {
    if a.abs_squared = Real.0 {
        a = Complex.0
    }
    if a = Complex.0 {
        a.abs_squared = Real.0
    }
}

/// A real number embeds as zero exactly when it is zero.
theorem from_real_eq_zero(a: Real) {
    Complex.from_real(a) = Complex.0 iff a = Real.0
} by {
    if Complex.from_real(a) = Complex.0 {
        a = Real.0
    }
    if a = Real.0 {
        Complex.from_real(a) = Complex.from_real(Real.0)
    }
}

/// Scalar multiplication of a complex number by a real number, acting on each component.
let complex_real_smul: Real -> Complex -> Complex = function(r: Real, c: Complex) {
    Complex.new(r * c.re, r * c.im)
}

/// Component-wise scalar distribution over scalar addition.
theorem complex_real_smul_add_left(r: Real, s: Real, x: Complex) {
    complex_real_smul(r + s, x) = complex_real_smul(r, x) + complex_real_smul(s, x)
} by {
    let lhs: Complex = complex_real_smul(r + s, x)
    let rhs: Complex = complex_real_smul(r, x) + complex_real_smul(s, x)
    complex_real_smul(s, x) = Complex.new(s * x.re, s * x.im)
    Complex.new(r * x.re, r * x.im).re = r * x.re
    Complex.new(r * x.re, r * x.im).im = r * x.im
    Complex.new(s * x.re, s * x.im).re = s * x.re
    Complex.new(s * x.re, s * x.im).im = s * x.im
    rhs.re = r * x.re + s * x.re
    rhs.im = r * x.im + s * x.im
    (r + s) * x.re = r * x.re + s * x.re
    (r + s) * x.im = r * x.im + s * x.im
    lhs.re = rhs.re
    lhs.im = rhs.im
    eq_by_components(lhs, rhs)
}

/// Component-wise scalar distribution over module addition.
theorem complex_real_smul_add_right(r: Real, x: Complex, y: Complex) {
    complex_real_smul(r, x + y) = complex_real_smul(r, x) + complex_real_smul(r, y)
} by {
    let lhs: Complex = complex_real_smul(r, x + y)
    let rhs: Complex = complex_real_smul(r, x) + complex_real_smul(r, y)
    (x + y).re = x.re + y.re
    (x + y).im = x.im + y.im
    lhs = Complex.new(r * (x + y).re, r * (x + y).im)
    Complex.new(r * (x + y).re, r * (x + y).im).re = r * (x + y).re
    Complex.new(r * (x + y).re, r * (x + y).im).im = r * (x + y).im
    complex_real_smul(r, x) = Complex.new(r * x.re, r * x.im)
    complex_real_smul(r, y) = Complex.new(r * y.re, r * y.im)
    Complex.new(r * x.re, r * x.im).re = r * x.re
    Complex.new(r * x.re, r * x.im).im = r * x.im
    Complex.new(r * y.re, r * y.im).re = r * y.re
    Complex.new(r * y.re, r * y.im).im = r * y.im
    rhs.re = r * x.re + r * y.re
    rhs.im = r * x.im + r * y.im
    r * (x.re + y.re) = r * x.re + r * y.re
    r * (x.im + y.im) = r * x.im + r * y.im
    lhs.re = rhs.re
    lhs.im = rhs.im
    eq_by_components(lhs, rhs)
}

/// Compatibility of the scalar action with real multiplication.
theorem complex_real_smul_assoc(r: Real, s: Real, x: Complex) {
    complex_real_smul(r * s, x) = complex_real_smul(r, complex_real_smul(s, x))
} by {
    complex_real_smul(r * s, x) = Complex.new((r * s) * x.re, (r * s) * x.im)
    (r * s) * x.re = r * (s * x.re)
    (r * s) * x.im = r * (s * x.im)
    complex_real_smul(s, x) = Complex.new(s * x.re, s * x.im)
    Complex.new(s * x.re, s * x.im).re = s * x.re
    Complex.new(s * x.re, s * x.im).im = s * x.im
    complex_real_smul(r, complex_real_smul(s, x)) = Complex.new(r * (s * x.re), r * (s * x.im))
}

/// The real one acts as the identity scalar on complex numbers.
theorem complex_real_smul_one(x: Complex) {
    complex_real_smul(Real.1, x) = x
} by {
    complex_real_smul(Real.1, x) = Complex.new(Real.1 * x.re, Real.1 * x.im)
    Real.1 * x.re = x.re
    Real.1 * x.im = x.im
    Complex.new(x.re, x.im) = x
}

/// The scalar action by zero produces the complex zero.
theorem complex_real_smul_zero_left(x: Complex) {
    complex_real_smul(Real.0, x) = Complex.0
} by {
    complex_real_smul(Real.0, x) = Complex.new(Real.0 * x.re, Real.0 * x.im)
    Real.0 * x.re = Real.0
    Real.0 * x.im = Real.0
    Complex.0 = Complex.new(Real.0, Real.0)
}

/// The scalar action on the complex zero produces the complex zero.
theorem complex_real_smul_zero_right(r: Real) {
    complex_real_smul(r, Complex.0) = Complex.0
} by {
    Complex.0.re = Real.0
    Complex.0.im = Real.0
    complex_real_smul(r, Complex.0) = Complex.new(r * Real.0, r * Real.0)
    r * Real.0 = Real.0
    Complex.0 = Complex.new(Real.0, Real.0)
}

/// The real part of a real-scalar product.
theorem re_smul(r: Real, x: Complex) {
    complex_real_smul(r, x).re = r * x.re
}

/// The imaginary part of a real-scalar product.
theorem im_smul(r: Real, x: Complex) {
    complex_real_smul(r, x).im = r * x.im
}

/// The embedding of the real zero is the complex zero.
theorem from_real_zero {
    Complex.from_real(Real.0) = Complex.0
}

/// The embedding of the real one is the complex one.
theorem from_real_one {
    Complex.from_real(Real.1) = Complex.1
}

/// The embedding of real numbers commutes with negation.
theorem from_real_neg(a: Real) {
    Complex.from_real(-a) = -Complex.from_real(a)
} by {
    Complex.from_real(-a) = Complex.new(-a, Real.0)
    Complex.from_real(a) = Complex.new(a, Real.0)
    (-Complex.from_real(a)).re = -Complex.from_real(a).re
    (-Complex.from_real(a)).im = -Complex.from_real(a).im
    Complex.from_real(a).re = a
    Complex.from_real(a).im = Real.0
    -Real.0 = Real.0
}

/// The embedding of real numbers commutes with subtraction.
theorem from_real_sub(a: Real, b: Real) {
    Complex.from_real(a - b) = Complex.from_real(a) - Complex.from_real(b)
} by {
    a - b = a + -b
    Complex.from_real(a + -b) = Complex.from_real(a) + Complex.from_real(-b)
    Complex.from_real(-b) = -Complex.from_real(b)
    Complex.from_real(a) + -Complex.from_real(b) = Complex.from_real(a) - Complex.from_real(b)
}

/// Inverse multiplied on the left of a nonzero complex number is the identity.
theorem inverse_mul_self(a: Complex) {
    a != Complex.0 implies a.inverse * a = Complex.1
} by {
    a * a.inverse = Complex.1
    a.inverse * a = a * a.inverse
}

/// Conjugation distributes over multiplication on the left.
theorem mul_conj_re(a: Complex) {
    (a * a.conj).re = a.abs_squared
} by {
    a * a.conj = Complex.new(a.abs_squared, Real.0)
}

/// The imaginary part of `a * a.conj` is zero.
theorem mul_conj_im(a: Complex) {
    (a * a.conj).im = Real.0
} by {
    a * a.conj = Complex.new(a.abs_squared, Real.0)
}

/// The product of a complex number with its conjugate is real.
theorem mul_conj_is_real(a: Complex) {
    (a * a.conj).is_real
} by {
    (a * a.conj).im = Real.0
}

/// The embedding of real division into the complex numbers.
theorem from_real_div(a: Real, b: Real) {
    Complex.from_real(a / b) = Complex.from_real(a) / Complex.from_real(b)
} by {
    Complex.from_real(a) / Complex.from_real(b) = Complex.from_real(a / b)
}

/// Conjugation commutes with inversion.
theorem conj_inverse(a: Complex) {
    a.inverse.conj = a.conj.inverse
} by {
    if a = Complex.0 {
        Complex.0.inverse = Complex.0
        Complex.0.conj = Complex.0
        a.inverse = Complex.0
        a.inverse.conj = Complex.0
        a.conj = Complex.0
        a.conj.inverse = Complex.0
    }
    if a != Complex.0 {
        a.conj.conj = a
        a.conj != Complex.0
        a.inverse = a.conj * Complex.from_real(a.abs_squared.inverse)
        a.inverse.conj = a.conj.conj * Complex.from_real(a.abs_squared.inverse).conj
        a.inverse.conj = a * Complex.from_real(a.abs_squared.inverse)
        a.conj.inverse = a.conj.conj * Complex.from_real(a.conj.abs_squared.inverse)
        a.conj.abs_squared = a.abs_squared
        a.conj.inverse = a * Complex.from_real(a.abs_squared.inverse)
    }
}

/// Conjugation distributes over division.
theorem conj_div(a: Complex, b: Complex) {
    (a / b).conj = a.conj / b.conj
} by {
    a / b = a * b.inverse
    (a * b.inverse).conj = a.conj * b.inverse.conj
    b.inverse.conj = b.conj.inverse
    a.conj * b.conj.inverse = a.conj / b.conj
}

/// The real-part function as a function on complex numbers.
let complex_re_fn: Complex -> Real = function(c: Complex) {
    c.re
}

/// The imaginary-part function as a function on complex numbers.
let complex_im_fn: Complex -> Real = function(c: Complex) {
    c.im
}

/// The real-part function preserves complex addition pointwise.
theorem complex_re_fn_add(a: Complex, b: Complex) {
    complex_re_fn(a + b) = complex_re_fn(a) + complex_re_fn(b)
} by {
    complex_re_fn(a + b) = (a + b).re
    (a + b).re = a.re + b.re
    complex_re_fn(a) = a.re
    complex_re_fn(b) = b.re
}

/// The imaginary-part function preserves complex addition pointwise.
theorem complex_im_fn_add(a: Complex, b: Complex) {
    complex_im_fn(a + b) = complex_im_fn(a) + complex_im_fn(b)
} by {
    complex_im_fn(a + b) = (a + b).im
    (a + b).im = a.im + b.im
    complex_im_fn(a) = a.im
    complex_im_fn(b) = b.im
}

/// The squared absolute value of a product with its inverse is one.
theorem abs_squared_mul_inverse(a: Complex) {
    a != Complex.0 implies a.abs_squared * a.inverse.abs_squared = Real.1
} by {
    if a != Complex.0 {
        a * a.inverse = Complex.1
        (a * a.inverse).abs_squared = a.abs_squared * a.inverse.abs_squared
        Complex.1.abs_squared = Real.1
        (a * a.inverse).abs_squared = Complex.1.abs_squared
    }
}
