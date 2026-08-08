from real import Real
from algebra.module.module import Module, ring_as_module, ring_as_module_smul
from algebra.module.module_hom import is_linear_map, preserves_add, preserves_smul, ModuleHom
from complex.complex import Complex, complex_re_fn, complex_im_fn, complex_re_fn_add, complex_im_fn_add,
    complex_real_smul, re_smul, im_smul
from complex.complex_module import complex_real_module, complex_real_module_smul

/// `complex_re_fn` preserves addition.
theorem complex_re_fn_preserves_add {
    preserves_add(complex_re_fn)
} by {
    forall(a: Complex, b: Complex) {
        complex_re_fn_add(a, b)
        complex_re_fn(a + b) = complex_re_fn(a) + complex_re_fn(b)
    }
}

/// `complex_im_fn` preserves addition.
theorem complex_im_fn_preserves_add {
    preserves_add(complex_im_fn)
} by {
    forall(a: Complex, b: Complex) {
        complex_im_fn_add(a, b)
        complex_im_fn(a + b) = complex_im_fn(a) + complex_im_fn(b)
    }
}

/// Helper: per-input form of the real-scalar preservation equality for `complex_re_fn`.
theorem complex_re_fn_smul_eq(r: Real, x: Complex) {
    complex_re_fn(complex_real_module.smul(r, x)) = ring_as_module[Real].smul(r, complex_re_fn(x))
} by {
    complex_real_module_smul(r, x)
    re_smul(r, x)
    ring_as_module_smul[Real](r, complex_re_fn(x))
}

/// `complex_re_fn` preserves the real scalar action.
theorem complex_re_fn_preserves_smul {
    preserves_smul(complex_real_module, ring_as_module[Real], complex_re_fn)
} by {
    forall(r: Real, x: Complex) {
        complex_re_fn_smul_eq(r, x)
    }
}

/// Helper: per-input form of the real-scalar preservation equality for `complex_im_fn`.
theorem complex_im_fn_smul_eq(r: Real, x: Complex) {
    complex_im_fn(complex_real_module.smul(r, x)) = ring_as_module[Real].smul(r, complex_im_fn(x))
} by {
    complex_real_module_smul(r, x)
    im_smul(r, x)
    ring_as_module_smul[Real](r, complex_im_fn(x))
}

/// `complex_im_fn` preserves the real scalar action.
theorem complex_im_fn_preserves_smul {
    preserves_smul(complex_real_module, ring_as_module[Real], complex_im_fn)
} by {
    forall(r: Real, x: Complex) {
        complex_im_fn_smul_eq(r, x)
    }
}

/// `complex_re_fn` is a real-linear map from the complex numbers to the real numbers.
theorem complex_re_fn_is_linear_map {
    is_linear_map(complex_real_module, ring_as_module[Real], complex_re_fn)
} by {
    complex_re_fn_preserves_add
    complex_re_fn_preserves_smul
}

/// `complex_im_fn` is a real-linear map from the complex numbers to the real numbers.
theorem complex_im_fn_is_linear_map {
    is_linear_map(complex_real_module, ring_as_module[Real], complex_im_fn)
} by {
    complex_im_fn_preserves_add
    complex_im_fn_preserves_smul
}

/// `ModuleHom.new` produces a `Some` value for the real-part function.
theorem complex_re_fn_module_hom_some {
    exists(h: ModuleHom[Real, Complex, Real]) {
        ModuleHom.new(complex_real_module, ring_as_module[Real], complex_re_fn) = Option.some(h)
    }
} by {
    complex_re_fn_is_linear_map
}

/// The real-part function packaged as a real-linear `ModuleHom`.
let complex_re_module_hom: ModuleHom[Real, Complex, Real] satisfy {
    ModuleHom.new(complex_real_module, ring_as_module[Real], complex_re_fn) = Option.some(complex_re_module_hom)
}

/// The source module of `complex_re_module_hom` is `complex_real_module`.
theorem complex_re_module_hom_src {
    complex_re_module_hom.src = complex_real_module
} by {
    ModuleHom.new(complex_real_module, ring_as_module[Real], complex_re_fn) = Option.some(complex_re_module_hom)
}

/// The destination module of `complex_re_module_hom` is `ring_as_module[Real]`.
theorem complex_re_module_hom_dst {
    complex_re_module_hom.dst = ring_as_module[Real]
} by {
    ModuleHom.new(complex_real_module, ring_as_module[Real], complex_re_fn) = Option.some(complex_re_module_hom)
}

/// `ModuleHom.new` produces a `Some` value for the imaginary-part function.
theorem complex_im_fn_module_hom_some {
    exists(h: ModuleHom[Real, Complex, Real]) {
        ModuleHom.new(complex_real_module, ring_as_module[Real], complex_im_fn) = Option.some(h)
    }
} by {
    complex_im_fn_is_linear_map
}

/// The imaginary-part function packaged as a real-linear `ModuleHom`.
let complex_im_module_hom: ModuleHom[Real, Complex, Real] satisfy {
    ModuleHom.new(complex_real_module, ring_as_module[Real], complex_im_fn) = Option.some(complex_im_module_hom)
}

/// The source module of `complex_im_module_hom` is `complex_real_module`.
theorem complex_im_module_hom_src {
    complex_im_module_hom.src = complex_real_module
} by {
    ModuleHom.new(complex_real_module, ring_as_module[Real], complex_im_fn) = Option.some(complex_im_module_hom)
}

/// The destination module of `complex_im_module_hom` is `ring_as_module[Real]`.
theorem complex_im_module_hom_dst {
    complex_im_module_hom.dst = ring_as_module[Real]
} by {
    ModuleHom.new(complex_real_module, ring_as_module[Real], complex_im_fn) = Option.some(complex_im_module_hom)
}

