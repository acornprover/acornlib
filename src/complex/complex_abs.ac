from real import Real
from real import sqrt_mul_self
from order import lte_antisymm
from real import mul_neg_left, mul_neg_right, mul_nonneg, square_nonneg
from real import sqrt_value_nonneg, square_le_square_of_nonneg
from complex.complex import Complex, abs_squared_nonneg, abs_squared_eq_zero, abs_squared_from_real,
    abs_squared_eq, abs_squared_mul, abs_squared_i, abs_squared_neg, im_conj, re_conj,
    from_real_neg, mul_i
from algebra.add_group import inverse_add, inverse_inverse

/// The modulus |z| of a complex number, the nonnegative square root of `abs_squared`.
let complex_modulus(z: Complex) -> r: Real satisfy {
    z.abs_squared.sqrt = Option.some(r) and r * r = z.abs_squared and r >= Real.0
} by {
    abs_squared_nonneg(z)
    sqrt_mul_self(z.abs_squared)
    let y: Real satisfy {
        z.abs_squared.sqrt = Option.some(y) and y * y = z.abs_squared
    }
    sqrt_value_nonneg(z.abs_squared, y)
}

attributes Complex {
    /// The modulus |z| of the complex number.
    let modulus: Complex -> Real = complex_modulus
}

/// The modulus is nonnegative.
theorem modulus_nonneg(z: Complex) {
    z.modulus >= Real.0
}

/// The square of the modulus equals `abs_squared`.
theorem modulus_squared(z: Complex) {
    z.modulus * z.modulus = z.abs_squared
}

/// Equal nonneg reals with equal squares are equal.
theorem nonneg_eq_of_squares_eq(a: Real, b: Real) {
    a >= Real.0 and b >= Real.0 and a * a = b * b implies a = b
} by {
    if a >= Real.0 and b >= Real.0 and a * a = b * b {
        square_le_square_of_nonneg(a, b)
        square_le_square_of_nonneg(b, a)
        b <= a
        lte_antisymm[Real](a, b)
        a = b
    }
}

/// The modulus is zero exactly when the complex number is zero.
theorem modulus_eq_zero(z: Complex) {
    z.modulus = Real.0 iff z = Complex.0
} by {
    if z.modulus = Real.0 {
        modulus_squared(z)
        z.abs_squared = Real.0
        abs_squared_eq_zero(z)
        z = Complex.0
    }
    if z = Complex.0 {
        abs_squared_eq_zero(z)
        modulus_squared(z)
        modulus_nonneg(z)
        nonneg_eq_of_squares_eq(z.modulus, Real.0)
        z.modulus = Real.0
    }
}

/// The modulus of zero is zero.
theorem modulus_of_zero {
    Complex.0.modulus = Real.0
} by {
    modulus_eq_zero(Complex.0)
}

/// Conjugation has the same `abs_squared`.
theorem conj_abs_squared(z: Complex) {
    z.conj.abs_squared = z.abs_squared
} by {
    abs_squared_eq(z.conj)
    mul_neg_left(z.im, -z.im)
    mul_neg_right(z.im, z.im)
    abs_squared_eq(z)
}

/// The modulus is multiplicative.
theorem modulus_mul(a: Complex, b: Complex) {
    (a * b).modulus = a.modulus * b.modulus
} by {
    abs_squared_mul(a, b)
    modulus_squared(a)
    modulus_squared(b)
    modulus_squared(a * b)
    modulus_nonneg(a)
    modulus_nonneg(b)
    modulus_nonneg(a * b)
    mul_nonneg(a.modulus, b.modulus)
    a.modulus * b.modulus >= Real.0
    nonneg_eq_of_squares_eq((a * b).modulus, a.modulus * b.modulus)
}

/// The modulus of one is one.
theorem modulus_of_one {
    Complex.1.modulus = Real.1
} by {
    abs_squared_eq(Complex.1)
    modulus_squared(Complex.1)
    modulus_nonneg(Complex.1)
    nonneg_eq_of_squares_eq(Complex.1.modulus, Real.1)
}

/// The modulus of a real number embedded in the complex plane equals its real absolute value.
theorem modulus_from_real_nonneg(a: Real) {
    a >= Real.0 implies Complex.from_real(a).modulus = a
} by {
    if a >= Real.0 {
        abs_squared_from_real(a)
        modulus_squared(Complex.from_real(a))
        Complex.from_real(a).modulus * Complex.from_real(a).modulus = a * a
        modulus_nonneg(Complex.from_real(a))
        nonneg_eq_of_squares_eq(Complex.from_real(a).modulus, a)
    }
}

/// The modulus of the imaginary unit is one.
theorem modulus_of_i {
    Complex.i.modulus = Real.1
} by {
    abs_squared_i
    modulus_squared(Complex.i)
    modulus_nonneg(Complex.i)
    Real.1 >= Real.0
    nonneg_eq_of_squares_eq(Complex.i.modulus, Real.1)
}

/// Negation preserves the modulus.
theorem modulus_neg(z: Complex) {
    (-z).modulus = z.modulus
} by {
    abs_squared_neg(z)
    modulus_squared(z)
    modulus_squared(-z)
    modulus_nonneg(z)
    modulus_nonneg(-z)
    nonneg_eq_of_squares_eq((-z).modulus, z.modulus)
}

/// The modulus of an embedded real number equals its absolute value.
theorem modulus_from_real(a: Real) {
    Complex.from_real(a).modulus = a.abs
} by {
    if a.is_negative {
        a.abs = -a
        from_real_neg(a)
        -a >= Real.0
        modulus_from_real_nonneg(-a)
        modulus_neg(Complex.from_real(a))
        Complex.from_real(a).modulus = -a
        Complex.from_real(a).modulus = a.abs
    } else {
        a >= Real.0
        modulus_from_real_nonneg(a)
        Complex.from_real(a).modulus = a.abs
    }
}

/// The modulus is preserved by conjugation.
theorem modulus_conj(z: Complex) {
    z.conj.modulus = z.modulus
} by {
    conj_abs_squared(z)
    modulus_squared(z)
    modulus_squared(z.conj)
    modulus_nonneg(z)
    modulus_nonneg(z.conj)
    nonneg_eq_of_squares_eq(z.conj.modulus, z.modulus)
}

/// The square of the real part is at most the squared modulus.
theorem re_sq_le_abs_squared(z: Complex) {
    z.re * z.re <= z.abs_squared
} by {
    abs_squared_eq(z)
    square_nonneg(z.im)
    z.re * z.re + Real.0 <= z.re * z.re + z.im * z.im
}

/// The real part is bounded above by the modulus.
theorem re_le_modulus(z: Complex) {
    z.re <= z.modulus
} by {
    modulus_nonneg(z)
    Real.0 <= z.modulus
    if z.re >= Real.0 {
        re_sq_le_abs_squared(z)
        modulus_squared(z)
        square_le_square_of_nonneg(z.re, z.modulus)
        z.re <= z.modulus
    } else {
        z.re <= Real.0
    }
}

/// The absolute value of the real part is bounded by the modulus.
theorem re_abs_le_modulus(z: Complex) {
    z.re.abs <= z.modulus
} by {
    if z.re.is_negative {
        z.re.abs = -z.re
        re_le_modulus(-z)
        modulus_neg(z)
        z.re.abs <= z.modulus
    } else {
        z.re.abs = z.re
        re_le_modulus(z)
        z.re.abs <= z.modulus
    }
}

/// The absolute value of the imaginary part is bounded by the modulus.
theorem im_abs_le_modulus(z: Complex) {
    z.im.abs <= z.modulus
} by {
    let w = Complex.i * z
    w.re = -z.im
    re_abs_le_modulus(w)
    modulus_mul(Complex.i, z)
    modulus_of_i
    w.modulus = z.modulus
}

/// The real part of `a * b.conj` equals the real inner product of components.
theorem re_mul_conj(a: Complex, b: Complex) {
    (a * b.conj).re = a.re * b.re + a.im * b.im
} by {
    b.conj = Complex.new(b.re, -b.im)
    re_conj(b)
    im_conj(b)
    b.conj.re = b.re
    b.conj.im = -b.im
    mul_neg_right(a.im, b.im)
    a.re * b.re - -(a.im * b.im) = a.re * b.re + a.im * b.im
}

/// Pure-arithmetic identity used to expand `|a+b|²` into `|a|² + |b|² + 2 Re(a conj b)`.
theorem abs_squared_add_identity(ar: Real, br: Real, ai: Real, bi: Real, p: Real, q: Real) {
    (ar + (p + p) + br) + (ai + (q + q) + bi) = (ar + ai) + (br + bi) + ((p + q) + (p + q))
} by {
    // Normalize both sides to ar + (ai + (br + (bi + (p + (p + (q + q))))))
    let lhs = (ar + (p + p) + br) + (ai + (q + q) + bi)
    ar + (p + p) + br = (ar + (p + p)) + br
    ai + (q + q) + bi = (ai + (q + q)) + bi
    lhs = (ar + (br + (p + p))) + (ai + (bi + (q + q)))
    (ai + (p + p)) + (bi + (q + q)) = ai + ((p + p) + (bi + (q + q)))
    (bi + (p + p)) + (q + q) = bi + ((p + p) + (q + q))
    p + (p + (q + q)) = p + (q + (p + q))
    (p + p) + (ai + (bi + (q + q))) = ai + (bi + ((p + q) + (p + q)))
    (br + (p + p)) + (ai + (bi + (q + q))) = br + (ai + (bi + ((p + q) + (p + q))))
    (ai + br) + (bi + ((p + q) + (p + q))) = ai + (br + (bi + ((p + q) + (p + q))))
    (ar + ai) + ((br + bi) + ((p + q) + (p + q))) = ((ar + ai) + (br + bi)) + ((p + q) + (p + q))
    ((ar + ai) + (br + bi)) + ((p + q) + (p + q)) = (ar + ai) + (br + bi) + ((p + q) + (p + q))
}

/// Expansion of the squared modulus of a sum.
theorem abs_squared_add(a: Complex, b: Complex) {
    (a + b).abs_squared = a.abs_squared + b.abs_squared + ((a * b.conj).re + (a * b.conj).re)
} by {
    abs_squared_eq(a + b)
    (a.re + b.re) * (a.re + b.re) = (a.re * a.re + a.re * b.re) + (a.re * b.re + b.re * b.re)
    (a.im + b.im) * (a.im + b.im) = (a.im * a.im + a.im * b.im) + (a.im * b.im + b.im * b.im)
    (a + b).abs_squared = (a.re * a.re + (a.re * b.re + a.re * b.re) + b.re * b.re) + (a.im * a.im + (a.im * b.im + a.im * b.im) + b.im * b.im)
    abs_squared_add_identity(a.re * a.re, b.re * b.re, a.im * a.im, b.im * b.im, a.re * b.re, a.im * b.im)
    abs_squared_eq(a)
    abs_squared_eq(b)
    re_mul_conj(a, b)
}

/// The triangle inequality for the complex modulus.
theorem modulus_triangle(a: Complex, b: Complex) {
    (a + b).modulus <= a.modulus + b.modulus
} by {
    let m = a.modulus + b.modulus
    modulus_nonneg(a)
    modulus_nonneg(b)
    a.modulus >= Real.0
    b.modulus >= Real.0
    Real.0 + b.modulus <= a.modulus + b.modulus
    Real.0 + b.modulus = b.modulus
    Real.0 <= a.modulus + b.modulus
    m >= Real.0

    let s = (a * b.conj).re
    re_le_modulus(a * b.conj)
    modulus_mul(a, b.conj)
    modulus_conj(b)
    s <= a.modulus * b.modulus
    s + s <= a.modulus * b.modulus + a.modulus * b.modulus

    abs_squared_add(a, b)
    a.abs_squared + b.abs_squared + (s + s) <= a.abs_squared + b.abs_squared + (a.modulus * b.modulus + a.modulus * b.modulus)

    modulus_squared(a)
    modulus_squared(b)

    (a.modulus + b.modulus) * (a.modulus + b.modulus) = (a.modulus * a.modulus + a.modulus * b.modulus) + (a.modulus * b.modulus + b.modulus * b.modulus)
    m * m = a.abs_squared + (a.modulus * b.modulus + a.modulus * b.modulus) + b.abs_squared
    let t = a.modulus * b.modulus + a.modulus * b.modulus
    a.abs_squared + (b.abs_squared + t) = (a.abs_squared + b.abs_squared) + t
    m * m = a.abs_squared + b.abs_squared + (a.modulus * b.modulus + a.modulus * b.modulus)

    modulus_squared(a + b)
    modulus_nonneg(a + b)
    square_le_square_of_nonneg((a + b).modulus, m)
    (a + b).modulus <= m
}

/// The modulus is bounded above by the sum of the absolute values of its coordinates.
theorem modulus_le_re_abs_add_im_abs(z: Complex) {
    z.modulus <= z.re.abs + z.im.abs
} by {
    let x = Complex.from_real(z.re)
    let y = Complex.i * Complex.from_real(z.im)
    mul_i(Complex.from_real(z.im))
    Complex.i * Complex.from_real(z.im) = Complex.new(Real.0, z.im)
    Complex.from_real(z.re).im = Real.0
    (Complex.i * Complex.from_real(z.im)).re = Real.0
    (Complex.i * Complex.from_real(z.im)).im = z.im
    z.re + Real.0 = z.re
    Real.0 + z.im = z.im
    Complex.from_real(z.re) + Complex.i * Complex.from_real(z.im) = Complex.new(z.re, z.im)
    modulus_triangle(x, y)
    modulus_from_real(z.re)
    modulus_mul(Complex.i, Complex.from_real(z.im))
    modulus_of_i
    modulus_from_real(z.im)
    Real.1 * z.im.abs = z.im.abs
}

/// The modulus is symmetric in the order of subtraction.
theorem modulus_sub_symm(a: Complex, b: Complex) {
    (a - b).modulus = (b - a).modulus
} by {
    let d = a - b
    let e = b - a
    d = a + -b
    inverse_add[Complex](a, -b)
    inverse_inverse[Complex](b)
    -d = b + -a
    e = -d
    modulus_neg(d)
}

/// The triangle inequality for a difference: |a - b| <= |a| + |b|.
theorem modulus_sub_triangle(a: Complex, b: Complex) {
    (a - b).modulus <= a.modulus + b.modulus
} by {
    modulus_triangle(a, -b)
    modulus_neg(b)
}

/// The reverse triangle inequality for the complex modulus.
theorem modulus_reverse_triangle(a: Complex, b: Complex) {
    (a.modulus - b.modulus).abs <= (a - b).modulus
} by {
    let d = a - b
    d = a + -b
    d + b = (a + -b) + b
    (a + -b) + b = a + (-b + b)
    -b + b = Complex.0
    a + (-b + b) = a + Complex.0
    a + Complex.0 = a
    d + b = a
    modulus_triangle(d, b)
    (d + b).modulus <= d.modulus + b.modulus
    a.modulus <= d.modulus + b.modulus
    a.modulus + -b.modulus <= (d.modulus + b.modulus) + -b.modulus
    (d.modulus + b.modulus) + -b.modulus = d.modulus + (b.modulus + -b.modulus)
    b.modulus + -b.modulus = Real.0
    d.modulus + (b.modulus + -b.modulus) = d.modulus + Real.0
    d.modulus + Real.0 = d.modulus
    a.modulus + -b.modulus = a.modulus - b.modulus
    a.modulus - b.modulus <= d.modulus

    let e = b - a
    e = b + -a
    b + (-a + a) = b + Complex.0
    e + a = b
    modulus_triangle(e, a)
    inverse_add[Complex](a, -b)
    inverse_inverse[Complex](b)
    modulus_neg(d)
    b.modulus <= d.modulus + a.modulus
    b.modulus + -a.modulus <= (d.modulus + a.modulus) + -a.modulus
    (d.modulus + a.modulus) + -a.modulus = d.modulus + (a.modulus + -a.modulus)
    d.modulus + (a.modulus + -a.modulus) = d.modulus + Real.0
    b.modulus + -a.modulus = b.modulus - a.modulus

    let diff = a.modulus - b.modulus
    if diff.is_negative {
        -(a.modulus + -b.modulus) = -a.modulus + -(-b.modulus)
        -a.modulus + b.modulus = b.modulus + -a.modulus
        -diff = b.modulus - a.modulus
        diff.abs = b.modulus - a.modulus
        diff.abs <= d.modulus
    } else {
        diff.abs = diff
        diff.abs <= d.modulus
    }
}
