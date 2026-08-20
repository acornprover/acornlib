from nat import Nat
from order import lte_trans, lte_lt_trans, lte_max_left, lte_max_right
from real import Real
from real import add_lt_lt, lt_trans
from real import converges_to, tail_bound, tail_bound_implies_is_close, eps_lt_half
from analysis import MetricSpace, tendsto_metric, metric_tail_bound,
    metric_tail_bound_at, metric_tail_bound_from_distances, tendsto_metric_tail_bound
from algebra.module.normed_add_comm_group import norm_distance
from complex.complex import re_sub, im_sub
from complex.complex_abs import re_abs_le_modulus, im_abs_le_modulus, modulus_le_re_abs_add_im_abs
from complex.complex_normed import Complex
from complex.complex_seq import complex_converges_to, componentwise_imp_complex_converges_to, re_seq, im_seq
from real import converges_to_has_tail_bound

/// The metric on `Complex` induced by its additive norm is the modulus of the difference.
instance Complex: MetricSpace {
    let distance: (Complex, Complex) -> Real = norm_distance[Complex]
}

/// The generic metric distance on `Complex` is the modulus of the difference.
/// This bridge pins the `MetricSpace` instance to the established modulus API and is used
/// in the convergence equivalence below.
theorem complex_distance_eq_modulus_sub(a: Complex, b: Complex) {
    a.distance(b) = (a - b).modulus
} by {
}

/// Componentwise convergence of a complex sequence implies convergence in the induced metric.
theorem complex_converges_to_imp_tendsto_metric(z: Nat -> Complex, w: Complex) {
    complex_converges_to(z, w) implies tendsto_metric(z, w)
} by {
    if complex_converges_to(z, w) {
        forall(eps: Real) {
            if eps.is_positive {
                eps_lt_half(eps)
                let half: Real satisfy {
                    half.is_positive and half + half < eps
                }
                converges_to_has_tail_bound(re_seq(z), w.re, half)
                let nr: Nat satisfy {
                    tail_bound(re_seq(z), w.re, nr, half)
                }
                converges_to_has_tail_bound(im_seq(z), w.im, half)
                let ni: Nat satisfy {
                    tail_bound(im_seq(z), w.im, ni, half)
                }
                let n = nr.max(ni)
                lte_max_left(nr, ni)
                lte_max_right(nr, ni)
                forall(i: Nat) {
                    if n <= i {
                        lte_trans(nr, n, i)
                        lte_trans(ni, n, i)
                        tail_bound_implies_is_close(re_seq(z), w.re, nr, half, i)
                        re_seq(z, i).is_close(w.re, half)
                        re_sub(z(i), w)
                        ((z(i) - w).re).abs < half
                        tail_bound_implies_is_close(im_seq(z), w.im, ni, half, i)
                        im_seq(z, i).is_close(w.im, half)
                        im_sub(z(i), w)
                        ((z(i) - w).im).abs < half
                        modulus_le_re_abs_add_im_abs(z(i) - w)
                        add_lt_lt(((z(i) - w).re).abs, half, ((z(i) - w).im).abs, half)
                        lte_lt_trans((z(i) - w).modulus, ((z(i) - w).re).abs + ((z(i) - w).im).abs, half + half)
                        lt_trans((z(i) - w).modulus, half + half, eps)
                        complex_distance_eq_modulus_sub(z(i), w)
                        z(i).distance(w) < eps
                    }
                }
                forall(i: Nat) {
                    n <= i implies z(i).distance(w) < eps
                }
                metric_tail_bound_from_distances(z, w, n, eps)
                exists(m: Nat) { metric_tail_bound(z, w, m, eps) }
            }
        }
    }
}

/// Metric convergence of a complex sequence forces convergence of the real-part sequence.
theorem tendsto_metric_imp_re_converges_to(z: Nat -> Complex, w: Complex) {
    tendsto_metric(z, w) implies converges_to(re_seq(z), w.re)
} by {
    if tendsto_metric(z, w) {
        forall(eps: Real) {
            if eps.is_positive {
                tendsto_metric_tail_bound(z, w, eps)
                let n: Nat satisfy {
                    metric_tail_bound(z, w, n, eps)
                }
                forall(i: Nat) {
                    if n <= i {
                        metric_tail_bound_at(z, w, n, eps, i)
                        complex_distance_eq_modulus_sub(z(i), w)
                        re_abs_le_modulus(z(i) - w)
                        lte_lt_trans(((z(i) - w).re).abs, (z(i) - w).modulus, eps)
                        ((z(i) - w).re).abs < eps
                        re_sub(z(i), w)
                        re_seq(z, i).is_close(w.re, eps)
                    }
                }
                forall(i: Nat) {
                    n <= i implies re_seq(z, i).is_close(w.re, eps)
                }
                tail_bound(re_seq(z), w.re, n, eps)
                exists(m: Nat) { tail_bound(re_seq(z), w.re, m, eps) }
            }
        }
    }
}

/// Metric convergence of a complex sequence forces convergence of the imaginary-part sequence.
theorem tendsto_metric_imp_im_converges_to(z: Nat -> Complex, w: Complex) {
    tendsto_metric(z, w) implies converges_to(im_seq(z), w.im)
} by {
    if tendsto_metric(z, w) {
        forall(eps: Real) {
            if eps.is_positive {
                tendsto_metric_tail_bound(z, w, eps)
                let n: Nat satisfy {
                    metric_tail_bound(z, w, n, eps)
                }
                forall(i: Nat) {
                    if n <= i {
                        metric_tail_bound_at(z, w, n, eps, i)
                        complex_distance_eq_modulus_sub(z(i), w)
                        im_abs_le_modulus(z(i) - w)
                        lte_lt_trans(((z(i) - w).im).abs, (z(i) - w).modulus, eps)
                        ((z(i) - w).im).abs < eps
                        im_sub(z(i), w)
                        im_seq(z, i).is_close(w.im, eps)
                    }
                }
                forall(i: Nat) {
                    n <= i implies im_seq(z, i).is_close(w.im, eps)
                }
                tail_bound(im_seq(z), w.im, n, eps)
                exists(m: Nat) { tail_bound(im_seq(z), w.im, m, eps) }
            }
        }
    }
}

/// Metric convergence of a complex sequence implies componentwise convergence.
theorem tendsto_metric_imp_complex_converges_to(z: Nat -> Complex, w: Complex) {
    tendsto_metric(z, w) implies complex_converges_to(z, w)
} by {
    if tendsto_metric(z, w) {
        tendsto_metric_imp_re_converges_to(z, w)
        tendsto_metric_imp_im_converges_to(z, w)
        converges_to(im_seq(z), w.im)
        componentwise_imp_complex_converges_to(z, w)
        complex_converges_to(z, w)
    }
}

/// Metric convergence of a complex sequence is exactly componentwise complex convergence.
theorem complex_tendsto_metric_iff_complex_converges_to(z: Nat -> Complex, w: Complex) {
    tendsto_metric(z, w) = complex_converges_to(z, w)
} by {
    if tendsto_metric(z, w) {
        tendsto_metric_imp_complex_converges_to(z, w)
        complex_converges_to(z, w)
    }
    if complex_converges_to(z, w) {
        complex_converges_to_imp_tendsto_metric(z, w)
        tendsto_metric(z, w)
    }
}
