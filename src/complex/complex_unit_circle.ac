// ---------------------------------------------------------------------------
// The unit circle.
//
// This module develops the elementary theory of the set of complex numbers of
// unit modulus:
//
//   |z| = 1  (the unit circle)
//
//   a. The unit circle is closed under multiplication, inversion and
//      conjugation; every element of the unit circle has its conjugate as
//      its inverse.
//   b. The special points 1, -1, i and -i lie on the unit circle, as do all
//      roots of unity and all exponentials exp(i·x) of purely imaginary
//      arguments.
// ---------------------------------------------------------------------------

from real import Real, two, pi
from nat import Nat
from algebra.field.field import unique_inverse
from complex.complex import Complex, mul_one_right, mul_one_left, from_real_one, from_real_zero
from complex.complex_abs import modulus_mul, modulus_conj, modulus_of_one, modulus_of_i,
    modulus_neg, modulus_nonneg, modulus_eq_zero
from complex.complex_exp_deep import complex_exp_unit_modulus
from complex.complex_roots_deep import omega_modulus
from complex.complex_log_deep import complex_modulus_inverse, complex_mul_conj_squared
from complex.complex_exp import complex_exp, complex_i_mul_real
from complex.roots_of_unity import omega

/// A complex number lies on the unit circle when |z| = 1.
define is_unit(z: Complex) -> Bool {
    z.modulus = Real.1
}

/// The product of two unit complex numbers is a unit complex number.
theorem unit_circle_mul(z: Complex, w: Complex) {
    is_unit(z) and is_unit(w) implies is_unit(z * w)
} by {
    if is_unit(z) and is_unit(w) {
        z.modulus = Real.1
        w.modulus = Real.1
        modulus_mul(z, w)
        (z * w).modulus = z.modulus * w.modulus
        z.modulus * w.modulus = Real.1 * Real.1
        Real.1 * Real.1 = Real.1
        (z * w).modulus = Real.1
        is_unit(z * w)
    }
}

/// The inverse of a unit complex number is a unit complex number.
theorem unit_circle_inverse(z: Complex) {
    is_unit(z) implies is_unit(z.inverse)
} by {
    if is_unit(z) {
        z.modulus = Real.1
        complex_modulus_inverse(z)
        z != Complex.0 implies z.inverse.modulus = z.modulus.inverse
        if z = Complex.0 {
            z.modulus = Real.1
            modulus_eq_zero(z)
            z.modulus = Real.0 iff z = Complex.0
            z.modulus = Real.0
            Real.1 = Real.0
            false
        }
        z != Complex.0
        z.inverse.modulus = z.modulus.inverse
        Real.1.inverse = Real.1
        z.modulus.inverse = Real.1
        z.inverse.modulus = Real.1
        is_unit(z.inverse)
    }
}

/// The conjugate of a unit complex number is a unit complex number.
theorem unit_circle_conj(z: Complex) {
    is_unit(z) implies is_unit(z.conj)
} by {
    if is_unit(z) {
        z.modulus = Real.1
        modulus_conj(z)
        z.conj.modulus = z.modulus
        z.conj.modulus = Real.1
        is_unit(z.conj)
    }
}

/// On the unit circle, the conjugate is the inverse:
/// conj(z) = z^(-1) for |z| = 1.
theorem unit_circle_conj_inverse(z: Complex) {
    is_unit(z) implies z.conj = z.inverse
} by {
    if is_unit(z) {
        z.modulus = Real.1
        complex_mul_conj_squared(z)
        z * z.conj = Complex.from_real(z.modulus * z.modulus)
        z.modulus * z.modulus = Real.1 * Real.1
        Real.1 * Real.1 = Real.1
        Complex.from_real(z.modulus * z.modulus) = Complex.from_real(Real.1)
        from_real_one
        Complex.from_real(Real.1) = Complex.1
        z * z.conj = Complex.1
        unique_inverse[Complex](z, z.conj)
        z * z.conj = Complex.1 implies z.conj = z.inverse
        z.conj = z.inverse
    }
}

/// One lies on the unit circle.
theorem unit_one {
    is_unit(Complex.1)
} by {
    modulus_of_one
    Complex.1.modulus = Real.1
    is_unit(Complex.1)
}

/// Negative one lies on the unit circle.
theorem unit_neg_one {
    is_unit(-Complex.1)
} by {
    modulus_neg(Complex.1)
    (-Complex.1).modulus = Complex.1.modulus
    modulus_of_one
    Complex.1.modulus = Real.1
    (-Complex.1).modulus = Real.1
    is_unit(-Complex.1)
}

/// The imaginary unit lies on the unit circle.
theorem unit_i {
    is_unit(Complex.i)
} by {
    modulus_of_i
    Complex.i.modulus = Real.1
    is_unit(Complex.i)
}

/// Negative the imaginary unit lies on the unit circle.
theorem unit_neg_i {
    is_unit(-Complex.i)
} by {
    modulus_neg(Complex.i)
    (-Complex.i).modulus = Complex.i.modulus
    modulus_of_i
    Complex.i.modulus = Real.1
    (-Complex.i).modulus = Real.1
    is_unit(-Complex.i)
}

/// Every root of unity lies on the unit circle: |omega(n)| = 1.
theorem unit_omega(n: Nat) {
    n >= Nat.1 implies is_unit(omega(n))
} by {
    if n >= Nat.1 {
        omega_modulus(n)
        n >= Nat.1 implies omega(n).modulus = Real.1
        omega(n).modulus = Real.1
        is_unit(omega(n))
    }
}

/// Every exponential of a purely imaginary number lies on the unit circle:
/// |exp(i·x)| = 1.
theorem unit_exp_i(x: Real) {
    is_unit(complex_exp(complex_i_mul_real(x)))
} by {
    complex_exp_unit_modulus(x)
    complex_exp(complex_i_mul_real(x)).modulus = Real.1
    is_unit(complex_exp(complex_i_mul_real(x)))
}

