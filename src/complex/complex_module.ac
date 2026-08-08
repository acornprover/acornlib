from real import Real
from complex.complex import Complex, complex_real_smul, complex_real_smul_add_left,
    complex_real_smul_add_right, complex_real_smul_assoc, complex_real_smul_one
from algebra.module.module import Module, is_module_action, module_smul_add_left_constraint,
    module_smul_add_right_constraint, module_smul_assoc_constraint,
    module_smul_one_constraint

/// `complex_real_smul` satisfies the left distributivity constraint of a module action.
theorem complex_real_smul_add_left_constraint {
    module_smul_add_left_constraint(complex_real_smul)
} by {
    forall(r: Real, s: Real, x: Complex) {
        complex_real_smul_add_left(r, s, x)
        complex_real_smul(r + s, x) = complex_real_smul(r, x) + complex_real_smul(s, x)
    }
}

/// `complex_real_smul` satisfies the right distributivity constraint of a module action.
theorem complex_real_smul_add_right_constraint {
    module_smul_add_right_constraint(complex_real_smul)
} by {
    forall(r: Real, x: Complex, y: Complex) {
        complex_real_smul_add_right(r, x, y)
        complex_real_smul(r, x + y) = complex_real_smul(r, x) + complex_real_smul(r, y)
    }
}

/// `complex_real_smul` satisfies the scalar-multiplication associativity constraint.
theorem complex_real_smul_assoc_constraint {
    module_smul_assoc_constraint(complex_real_smul)
} by {
    forall(r: Real, s: Real, x: Complex) {
        complex_real_smul_assoc(r, s, x)
        complex_real_smul(r * s, x) = complex_real_smul(r, complex_real_smul(s, x))
    }
}

/// `complex_real_smul` satisfies the unital constraint of a module action.
theorem complex_real_smul_one_constraint {
    module_smul_one_constraint(complex_real_smul)
} by {
    forall(x: Complex) {
        complex_real_smul_one(x)
        complex_real_smul(Real.1, x) = x
    }
}

/// `complex_real_smul` is a left module action of the reals on the complex numbers.
theorem complex_real_smul_is_module_action {
    is_module_action(complex_real_smul)
} by {
    complex_real_smul_add_left_constraint
    complex_real_smul_add_right_constraint
    complex_real_smul_assoc_constraint
    complex_real_smul_one_constraint
}

/// The complex numbers as a left module over the real numbers, with scalar action
/// `complex_real_smul`.
let complex_real_module: Module[Real, Complex] satisfy {
    Module.new(complex_real_smul) = Option.some(complex_real_module)
}

/// The scalar action on the complex-as-real-module is `complex_real_smul`.
theorem complex_real_module_smul(r: Real, x: Complex) {
    complex_real_module.smul(r, x) = complex_real_smul(r, x)
} by {
}
