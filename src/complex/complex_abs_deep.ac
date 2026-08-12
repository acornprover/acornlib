// ---------------------------------------------------------------------------
// Deepening of the complex modulus theory.
//
// This module extends src/complex/complex_abs.ac with the further identities
// of the modulus |z|:
//
//   a. Positivity: |z| > 0 for z != 0, and |z| != 0  iff  z != 0.
//   b. Multiplicativity variants: |z·w|, |conj(z)·w|, |z·conj(w)|, |a·b·c|,
//      |i·z|, |(-i)·z|, |2·z|, |r·z| (r real), |i^n| and |(-1)^n|.
//   c. The triangle inequality variants: |a + b + c|, |a - b| <= |a - c| +
//      |c - b|, |a| - |b| bounded by |a + b|, and |a| <= |a + b| + |b|.
//   d. Squared-modulus identities: |z|² = Re(z)² + Im(z)², |z|² = abs_squared,
//      the square bound of the triangle inequality, and |conj(z)|² = |z|².
// ---------------------------------------------------------------------------

from real import Real, two, pos_imp_eq_abs, real_mul_comm, mul_pos_pos, add_lte_add,
    mul_le_mul_of_nonneg_left, mul_le_mul_of_nonneg_right
from order import lte_trans
from nat import Nat, one_pow
from complex.complex import Complex, mul_comm, mul_assoc, re_from_real, im_from_real,
    add_zero_right, add_zero_left, mul_one_left, add_comm, real_mul_lifts, from_real_one,
    eq_by_components, abs_squared_eq, abs_squared_mul, add_assoc
from complex.complex_abs import modulus_mul, modulus_conj, modulus_eq_zero, modulus_of_one,
    modulus_from_real, modulus_nonneg, modulus_neg, modulus_of_i, modulus_triangle,
    modulus_squared, modulus_sub_triangle, modulus_reverse_triangle, modulus_of_zero,
    nonneg_eq_of_squares_eq
from complex.complex_algebra_deep import sub_add_telescope, sub_neg_eq_add, add_sub_cancel,
    sub_self_restated, neg_neg_complex
from complex.complex_exp import complex_modulus_pow, real_two_positive, real_mul_one_left
from complex.complex_trig import complex_neg_zero
from complex.complex_properties import pow_two_neg_one, pow_two_i

// ---------------------------------------------------------------------------
// a. Positivity of the modulus.
// ---------------------------------------------------------------------------

/// A nonzero complex number has positive modulus: |z| > 0.
theorem modulus_pos(z: Complex) {
    z != Complex.0 implies z.modulus > Real.0
} by {
    if z != Complex.0 {
        modulus_eq_zero(z)
        z.modulus = Real.0 iff z = Complex.0
        z.modulus != Real.0
        modulus_nonneg(z)
        z.modulus >= Real.0
        z.modulus > Real.0
    }
}

/// The modulus is nonzero exactly for nonzero complex numbers.
theorem modulus_nonzero_iff(z: Complex) {
    z.modulus != Real.0 iff z != Complex.0
} by {
    if z.modulus != Real.0 {
        if z = Complex.0 {
            modulus_eq_zero(z)
            z.modulus = Real.0 iff z = Complex.0
            z.modulus = Real.0
            false
        }
        z != Complex.0
    }
    if z != Complex.0 {
        if z.modulus = Real.0 {
            modulus_eq_zero(z)
            z.modulus = Real.0 iff z = Complex.0
            z = Complex.0
            false
        }
        z.modulus != Real.0
    }
}

/// The squared modulus of a nonzero complex number is positive.
theorem modulus_sq_pos(z: Complex) {
    z != Complex.0 implies z.modulus * z.modulus > Real.0
} by {
    if z != Complex.0 {
        modulus_pos(z)
        z.modulus > Real.0
        mul_pos_pos(z.modulus, z.modulus)
        z.modulus > Real.0 and z.modulus > Real.0 implies (z.modulus * z.modulus).is_positive
        (z.modulus * z.modulus).is_positive
        (z.modulus * z.modulus) > Real.0
    }
}

// ---------------------------------------------------------------------------
// b. Multiplicativity variants.
// ---------------------------------------------------------------------------

/// The modulus of a square is the square of the modulus: |z²| = |z|².
theorem modulus_pow_two(z: Complex) {
    (z * z).modulus = z.modulus * z.modulus
} by {
    modulus_mul(z, z)
    (z * z).modulus = z.modulus * z.modulus
}

/// The modulus of a triple product: |a·b·c| = |a|·|b|·|c|.
theorem modulus_of_three(a: Complex, b: Complex, c: Complex) {
    (a * b * c).modulus = a.modulus * b.modulus * c.modulus
} by {
    (a * b * c).modulus = ((a * b) * c).modulus
    modulus_mul(a * b, c)
    ((a * b) * c).modulus = (a * b).modulus * c.modulus
    modulus_mul(a, b)
    (a * b).modulus = a.modulus * b.modulus
    (a * b * c).modulus = a.modulus * b.modulus * c.modulus
}

/// The modulus of conj(z)·w is |z|·|w|.
theorem modulus_of_conj_mul(a: Complex, b: Complex) {
    (a.conj * b).modulus = a.modulus * b.modulus
} by {
    modulus_mul(a.conj, b)
    (a.conj * b).modulus = a.conj.modulus * b.modulus
    modulus_conj(a)
    a.conj.modulus = a.modulus
    (a.conj * b).modulus = a.modulus * b.modulus
}

/// The modulus of z·conj(w) is |z|·|w|.
theorem modulus_of_mul_conj(a: Complex, b: Complex) {
    (a * b.conj).modulus = a.modulus * b.modulus
} by {
    modulus_mul(a, b.conj)
    (a * b.conj).modulus = a.modulus * b.conj.modulus
    modulus_conj(b)
    b.conj.modulus = b.modulus
    (a * b.conj).modulus = a.modulus * b.modulus
}

/// Multiplication by i preserves the modulus: |i·z| = |z|.
theorem modulus_of_i_mul(z: Complex) {
    (Complex.i * z).modulus = z.modulus
} by {
    modulus_mul(Complex.i, z)
    (Complex.i * z).modulus = Complex.i.modulus * z.modulus
    modulus_of_i
    Complex.i.modulus = Real.1
    Complex.i.modulus * z.modulus = Real.1 * z.modulus
    real_mul_one_left(z.modulus)
    Real.1 * z.modulus = z.modulus
    (Complex.i * z).modulus = z.modulus
}

/// Multiplication by -i preserves the modulus: |(-i)·z| = |z|.
theorem modulus_of_neg_i_mul(z: Complex) {
    ((-Complex.i) * z).modulus = z.modulus
} by {
    modulus_mul(-Complex.i, z)
    ((-Complex.i) * z).modulus = (-Complex.i).modulus * z.modulus
    modulus_neg(Complex.i)
    (-Complex.i).modulus = Complex.i.modulus
    modulus_of_i
    Complex.i.modulus = Real.1
    (-Complex.i).modulus * z.modulus = Real.1 * z.modulus
    real_mul_one_left(z.modulus)
    Real.1 * z.modulus = z.modulus
    ((-Complex.i) * z).modulus = z.modulus
}

/// The modulus of twice a number: |2·z| = 2·|z|.
theorem modulus_double(z: Complex) {
    (Complex.from_real(two) * z).modulus = two * z.modulus
} by {
    modulus_mul(Complex.from_real(two), z)
    (Complex.from_real(two) * z).modulus = Complex.from_real(two).modulus * z.modulus
    modulus_from_real(two)
    Complex.from_real(two).modulus = two.abs
    real_two_positive
    two > Real.0
    two.is_positive
    pos_imp_eq_abs(two)
    two = two.abs
    two.abs = two
    Complex.from_real(two).modulus = two
    Complex.from_real(two).modulus * z.modulus = two * z.modulus
    (Complex.from_real(two) * z).modulus = two * z.modulus
}

/// The modulus of a real scalar multiple: |r·z| = |r|·|z|.
theorem modulus_of_smul_real(r: Real, z: Complex) {
    (Complex.from_real(r) * z).modulus = r.abs * z.modulus
} by {
    modulus_mul(Complex.from_real(r), z)
    (Complex.from_real(r) * z).modulus = Complex.from_real(r).modulus * z.modulus
    modulus_from_real(r)
    Complex.from_real(r).modulus = r.abs
    Complex.from_real(r).modulus * z.modulus = r.abs * z.modulus
    (Complex.from_real(r) * z).modulus = r.abs * z.modulus
}

/// The modulus of every power of i is one: |i^n| = 1.
theorem modulus_of_pow_i(n: Nat) {
    Complex.i.pow(n).modulus = Real.1
} by {
    complex_modulus_pow(Complex.i, n)
    Complex.i.pow(n).modulus = Complex.i.modulus.pow(n)
    modulus_of_i
    Complex.i.modulus = Real.1
    Complex.i.pow(n).modulus = Real.1.pow(n)
    one_pow[Real](n)
    Real.1.pow(n) = Real.1
    Complex.i.pow(n).modulus = Real.1
}

/// The modulus of every power of -1 is one: |(-1)^n| = 1.
theorem modulus_of_pow_neg_one(n: Nat) {
    (-Complex.1).pow(n).modulus = Real.1
} by {
    complex_modulus_pow(-Complex.1, n)
    (-Complex.1).pow(n).modulus = (-Complex.1).modulus.pow(n)
    modulus_neg(Complex.1)
    (-Complex.1).modulus = Complex.1.modulus
    modulus_of_one
    Complex.1.modulus = Real.1
    (-Complex.1).pow(n).modulus = Real.1.pow(n)
    one_pow[Real](n)
    Real.1.pow(n) = Real.1
    (-Complex.1).pow(n).modulus = Real.1
}

/// The modulus of a product is independent of the order of the factors.
theorem modulus_mul_comm_abs(a: Complex, b: Complex) {
    (a * b).modulus = (b * a).modulus
} by {
    modulus_mul(a, b)
    (a * b).modulus = a.modulus * b.modulus
    modulus_mul(b, a)
    (b * a).modulus = b.modulus * a.modulus
    real_mul_comm(a.modulus, b.modulus)
    a.modulus * b.modulus = b.modulus * a.modulus
    (a * b).modulus = (b * a).modulus
}

// ---------------------------------------------------------------------------
// c. The triangle inequality variants.
// ---------------------------------------------------------------------------

/// The triangle inequality for three summands:
/// |a + b + c| <= |a| + |b| + |c|.
theorem modulus_triangle_three(a: Complex, b: Complex, c: Complex) {
    (a + b + c).modulus <= a.modulus + b.modulus + c.modulus
} by {
    (a + b + c).modulus = ((a + b) + c).modulus
    modulus_triangle(a + b, c)
    ((a + b) + c).modulus <= (a + b).modulus + c.modulus
    modulus_triangle(a, b)
    (a + b).modulus <= a.modulus + b.modulus
    add_lte_add((a + b).modulus, a.modulus + b.modulus, c.modulus, c.modulus)
    (a + b).modulus + c.modulus <= a.modulus + b.modulus + c.modulus
    lte_trans(((a + b) + c).modulus, (a + b).modulus + c.modulus, a.modulus + b.modulus + c.modulus)
    ((a + b) + c).modulus <= a.modulus + b.modulus + c.modulus
    (a + b + c).modulus <= a.modulus + b.modulus + c.modulus
}

/// The triangle inequality through an intermediate point:
/// |a - b| <= |a - c| + |c - b|.
theorem modulus_triangle_sub_add(a: Complex, b: Complex, c: Complex) {
    (a - b).modulus <= (a - c).modulus + (c - b).modulus
} by {
    sub_add_telescope(a, c, b)
    (a - c) + (c - b) = a - b
    modulus_triangle(a - c, c - b)
    ((a - c) + (c - b)).modulus <= (a - c).modulus + (c - b).modulus
    (a - b).modulus <= (a - c).modulus + (c - b).modulus
}

/// The reverse triangle inequality for sums:
/// | |a| - |b| | <= |a + b|.
theorem modulus_add_reverse(a: Complex, b: Complex) {
    (a.modulus - b.modulus).abs <= (a + b).modulus
} by {
    modulus_reverse_triangle(a, -b)
    (a.modulus - (-b).modulus).abs <= (a - (-b)).modulus
    modulus_neg(b)
    (-b).modulus = b.modulus
    (a.modulus - (-b).modulus).abs = (a.modulus - b.modulus).abs
    sub_neg_eq_add(a, b)
    a - (-b) = a + b
    (a - (-b)).modulus = (a + b).modulus
    (a.modulus - b.modulus).abs <= (a + b).modulus
}

/// The modulus of a summand is bounded by the modulus of the sum plus the
/// modulus of the other summand: |a| <= |a + b| + |b|.
theorem modulus_le_add(a: Complex, b: Complex) {
    a.modulus <= (a + b).modulus + b.modulus
} by {
    add_sub_cancel(a, b)
    (a + b) - b = a
    modulus_sub_triangle(a + b, b)
    ((a + b) - b).modulus <= (a + b).modulus + b.modulus
    a.modulus <= (a + b).modulus + b.modulus
}

/// The modulus of a difference with the zero element: |z - 0| = |z|.
theorem modulus_sub_zero(z: Complex) {
    (z - Complex.0).modulus = z.modulus
} by {
    z - Complex.0 = z + -Complex.0
    complex_neg_zero
    -Complex.0 = Complex.0
    z - Complex.0 = z + Complex.0
    add_zero_right(z)
    z + Complex.0 = z
    (z - Complex.0).modulus = z.modulus
}

/// The modulus of zero minus z: |0 - z| = |z|.
theorem modulus_zero_sub(z: Complex) {
    (Complex.0 - z).modulus = z.modulus
} by {
    Complex.0 - z = Complex.0 + -z
    add_zero_left(-z)
    Complex.0 + -z = -z
    modulus_neg(z)
    (-z).modulus = z.modulus
    (Complex.0 - z).modulus = z.modulus
}

/// The modulus of a self-difference is zero: |z - z| = 0.
theorem modulus_sub_self_zero(z: Complex) {
    (z - z).modulus = Real.0
} by {
    sub_self_restated(z)
    z - z = Complex.0
    (z - z).modulus = Complex.0.modulus
    modulus_of_zero
    Complex.0.modulus = Real.0
    (z - z).modulus = Real.0
}

/// The modulus of the conjugate of a difference is the modulus of the difference.
theorem modulus_of_conj_sub(a: Complex, b: Complex) {
    (a - b).conj.modulus = (a - b).modulus
} by {
    modulus_conj(a - b)
    (a - b).conj.modulus = (a - b).modulus
}

/// The triangle inequality applied to a sum of products:
/// |a·b + c·d| <= |a|·|b| + |c|·|d|.
theorem modulus_add_mul_bound(a: Complex, b: Complex, c: Complex, d: Complex) {
    (a * b + c * d).modulus <= a.modulus * b.modulus + c.modulus * d.modulus
} by {
    modulus_triangle(a * b, c * d)
    (a * b + c * d).modulus <= (a * b).modulus + (c * d).modulus
    modulus_mul(a, b)
    (a * b).modulus = a.modulus * b.modulus
    modulus_mul(c, d)
    (c * d).modulus = c.modulus * d.modulus
    (a * b).modulus + (c * d).modulus = a.modulus * b.modulus + c.modulus * d.modulus
    (a * b + c * d).modulus <= a.modulus * b.modulus + c.modulus * d.modulus
}

// ---------------------------------------------------------------------------
// d. Squared-modulus identities.
// ---------------------------------------------------------------------------

/// The squared modulus equals the sum of the squared coordinates:
/// |z|² = Re(z)² + Im(z)².
theorem modulus_sq_re_im(z: Complex) {
    z.modulus * z.modulus = z.re * z.re + z.im * z.im
} by {
    modulus_squared(z)
    z.modulus * z.modulus = z.abs_squared
    abs_squared_eq(z)
    z.abs_squared = z.re * z.re + z.im * z.im
    z.modulus * z.modulus = z.re * z.re + z.im * z.im
}

/// The squared modulus is the squared absolute value:
/// |z|² = abs_squared(z).
theorem abs_squared_of_modulus(z: Complex) {
    z.abs_squared = z.modulus * z.modulus
} by {
    modulus_squared(z)
    z.modulus * z.modulus = z.abs_squared
    z.abs_squared = z.modulus * z.modulus
}

/// The squared modulus of the conjugate is the squared modulus:
/// |conj(z)|² = |z|².
theorem modulus_conj_sq_eq(z: Complex) {
    z.conj.modulus * z.conj.modulus = z.modulus * z.modulus
} by {
    modulus_conj(z)
    z.conj.modulus = z.modulus
    z.conj.modulus * z.conj.modulus = z.modulus * z.modulus
}

/// The square of the triangle inequality:
/// |a + b|² <= (|a| + |b|)².
theorem modulus_add_sq_bound(a: Complex, b: Complex) {
    (a + b).modulus * (a + b).modulus <= (a.modulus + b.modulus) * (a.modulus + b.modulus)
} by {
    modulus_triangle(a, b)
    (a + b).modulus <= a.modulus + b.modulus
    modulus_nonneg(a + b)
    (a + b).modulus >= Real.0
    modulus_nonneg(a)
    a.modulus >= Real.0
    Real.0 <= a.modulus
    modulus_nonneg(b)
    b.modulus >= Real.0
    Real.0 <= b.modulus
    add_lte_add(Real.0, a.modulus, Real.0, b.modulus)
    Real.0 <= a.modulus and Real.0 <= b.modulus implies Real.0 + Real.0 <= a.modulus + b.modulus
    Real.0 + Real.0 <= a.modulus + b.modulus
    a.modulus + b.modulus >= Real.0
    mul_le_mul_of_nonneg_left[Real]((a + b).modulus, a.modulus + b.modulus, (a + b).modulus)
    (a + b).modulus <= a.modulus + b.modulus and Real.0 <= (a + b).modulus implies (a + b).modulus * (a + b).modulus <= (a + b).modulus * (a.modulus + b.modulus)
    (a + b).modulus * (a + b).modulus <= (a + b).modulus * (a.modulus + b.modulus)
    mul_le_mul_of_nonneg_right[Real]((a + b).modulus, a.modulus + b.modulus, a.modulus + b.modulus)
    (a + b).modulus <= a.modulus + b.modulus and Real.0 <= a.modulus + b.modulus implies (a + b).modulus * (a.modulus + b.modulus) <= (a.modulus + b.modulus) * (a.modulus + b.modulus)
    (a + b).modulus * (a.modulus + b.modulus) <= (a.modulus + b.modulus) * (a.modulus + b.modulus)
    lte_trans((a + b).modulus * (a + b).modulus,
        (a + b).modulus * (a.modulus + b.modulus),
        (a.modulus + b.modulus) * (a.modulus + b.modulus))
    (a + b).modulus * (a + b).modulus <= (a.modulus + b.modulus) * (a.modulus + b.modulus)
}
