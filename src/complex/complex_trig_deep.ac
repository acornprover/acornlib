// ---------------------------------------------------------------------------
// Deepening of the complex trigonometric functions.
//
// This module extends src/complex/complex_trig.ac with the fundamental
// identities of the complex sine and cosine:
//
//   a. Parity: (-z).cos = z.cos and (-z).sin = -z.sin.
//   b. Euler's formula for complex arguments: exp(i·z) = z.cos + i·z.sin,
//      and exp(-i·z) = z.cos - i·z.sin.
//   c. The Pythagorean identity (z).cos² + (z).sin² = 1.
//   d. The addition formulas for sine and cosine, and the double-angle
//      formulas.
//   e. Periodicity: (z + 2·pi).sin = z.sin and (z + 2·pi).cos = z.cos.
//   f. The special values pi.sin = 0 and pi.cos = -1.
// ---------------------------------------------------------------------------

from real import Real, two, pi, cos_pi_neg_one, sin_pi_zero
from algebra.field.field import unique_inverse
from complex.complex import Complex, distrib, mul_comm, mul_assoc, add_assoc, add_comm,
    add_zero_left, add_zero_right, mul_one_left, mul_one_right, mul_zero_left,
    i_squared_eq_neg_one, neg_eq_mul_neg_one, eq_by_components, re_add, im_add, left_distrib,
    real_add_lifts, from_real_one, from_real_neg, from_real_zero, mul_inverse,
    real_mul_lifts
from algebra.ring.ring import mul_sub_left, mul_neg_left, mul_neg_neg, mul_neg_right
from complex.complex_algebra_deep import neg_mul_right, neg_add, neg_sub, neg_neg_complex,
    sub_self_restated, add_sub_cancel, two_eq_real_one_plus_one, inverse_one_restated,
    complex_add_sub_cancel_twice, div_distrib_right, neg_div_num, mul_conj_expansion
from complex.complex_trig import complex_cos, complex_sin, complex_cos_from_real,
    complex_sin_from_real, complex_from_real_two_nonzero, complex_two_i_nonzero,
    complex_div_mul_cancel, complex_new_eq_from_real_i
from complex.complex_exp import complex_exp, complex_exp_add, complex_i_mul_real,
    complex_neg_one_new
from complex.complex_log_deep import complex_exp_neg, complex_exp_mul_neg
from complex.roots_of_unity import complex_exp_two_pi

// ---------------------------------------------------------------------------
// Local arithmetic helpers.
// ---------------------------------------------------------------------------

/// Doubling a complex number: 2·z = z + z.
theorem complex_two_mul(a: Complex) {
    Complex.from_real(two) * a = a + a
} by {
    two_eq_real_one_plus_one
    two = Real.1 + Real.1
    Complex.from_real(two) = Complex.from_real(Real.1 + Real.1)
    real_add_lifts(Real.1, Real.1)
    Complex.from_real(Real.1 + Real.1) = Complex.from_real(Real.1) + Complex.from_real(Real.1)
    from_real_one
    Complex.from_real(Real.1) = Complex.1
    Complex.from_real(two) = Complex.1 + Complex.1
    Complex.from_real(two) * a = (Complex.1 + Complex.1) * a
    left_distrib(Complex.1, Complex.1, a)
    (Complex.1 + Complex.1) * a = Complex.1 * a + Complex.1 * a
    mul_one_left(a)
    Complex.1 * a = a
    (Complex.1 + Complex.1) * a = a + a
    Complex.from_real(two) * a = a + a
}

/// (a + b) + (a - b) = 2·a.
theorem complex_add_sub_same(a: Complex, b: Complex) {
    (a + b) + (a - b) = Complex.from_real(two) * a
} by {
    add_assoc(a, b, a - b)
    (a + b) + (a - b) = a + (b + (a - b))
    b + (a - b) = b + (a + -b)
    add_assoc(b, a, -b)
    b + (a + -b) = (b + a) + -b
    add_comm(b, a)
    b + a = a + b
    (b + a) + -b = (a + b) + -b
    add_sub_cancel(a, b)
    (a + b) - b = a
    (a + b) + -b = a
    b + (a - b) = a
    (a + b) + (a - b) = a + a
    complex_two_mul(a)
    Complex.from_real(two) * a = a + a
    (a + b) + (a - b) = Complex.from_real(two) * a
}

/// The imaginary unit divided by two times itself is one half:
/// i·(2·i)^(-1) = 2^(-1).
theorem complex_i_over_two_i {
    Complex.i * (Complex.from_real(two) * Complex.i).inverse = Complex.from_real(two).inverse
} by {
    mul_assoc(Complex.from_real(two), Complex.i, (Complex.from_real(two) * Complex.i).inverse)
    (Complex.from_real(two) * Complex.i) * (Complex.from_real(two) * Complex.i).inverse = Complex.from_real(two) * (Complex.i * (Complex.from_real(two) * Complex.i).inverse)
    complex_two_i_nonzero
    Complex.from_real(two) * Complex.i != Complex.0
    mul_inverse(Complex.from_real(two) * Complex.i)
    Complex.from_real(two) * Complex.i != Complex.0 implies (Complex.from_real(two) * Complex.i) * (Complex.from_real(two) * Complex.i).inverse = Complex.1
    (Complex.from_real(two) * Complex.i) * (Complex.from_real(two) * Complex.i).inverse = Complex.1
    Complex.from_real(two) * (Complex.i * (Complex.from_real(two) * Complex.i).inverse) = Complex.1
    unique_inverse[Complex](Complex.from_real(two), Complex.i * (Complex.from_real(two) * Complex.i).inverse)
    Complex.from_real(two) * (Complex.i * (Complex.from_real(two) * Complex.i).inverse) = Complex.1 implies Complex.i * (Complex.from_real(two) * Complex.i).inverse = Complex.from_real(two).inverse
    Complex.i * (Complex.from_real(two) * Complex.i).inverse = Complex.from_real(two).inverse
}

/// The expansion (a + i·b)·(c + i·d) = (a·c - b·d) + i·(a·d + b·c).
theorem complex_mul_i_forms(a: Complex, b: Complex, c: Complex, d: Complex) {
    (a + Complex.i * b) * (c + Complex.i * d) = (a * c - b * d) + Complex.i * (a * d + b * c)
} by {
    distrib(a, Complex.i * b, c + Complex.i * d)
    (a + Complex.i * b) * (c + Complex.i * d) = a * (c + Complex.i * d) + (Complex.i * b) * (c + Complex.i * d)
    distrib(a, c, Complex.i * d)
    a * (c + Complex.i * d) = a * c + a * (Complex.i * d)
    distrib(Complex.i * b, c, Complex.i * d)
    (Complex.i * b) * (c + Complex.i * d) = (Complex.i * b) * c + (Complex.i * b) * (Complex.i * d)
    (a + Complex.i * b) * (c + Complex.i * d) = a * c + a * (Complex.i * d) + (Complex.i * b) * c + (Complex.i * b) * (Complex.i * d)
    mul_comm(a, Complex.i * d)
    a * (Complex.i * d) = (Complex.i * d) * a
    mul_assoc(Complex.i, d, a)
    (Complex.i * d) * a = Complex.i * (d * a)
    mul_comm(d, a)
    d * a = a * d
    a * (Complex.i * d) = Complex.i * (a * d)
    mul_assoc(Complex.i, b, c)
    (Complex.i * b) * c = Complex.i * (b * c)
    mul_comm(Complex.i, b)
    Complex.i * b = b * Complex.i
    (Complex.i * b) * (Complex.i * d) = (b * Complex.i) * (Complex.i * d)
    mul_assoc(b, Complex.i, Complex.i * d)
    (b * Complex.i) * (Complex.i * d) = b * (Complex.i * (Complex.i * d))
    mul_assoc(Complex.i, Complex.i, d)
    Complex.i * (Complex.i * d) = (Complex.i * Complex.i) * d
    i_squared_eq_neg_one
    Complex.i * Complex.i = -Complex.1
    (Complex.i * Complex.i) * d = (-Complex.1) * d
    neg_eq_mul_neg_one(d)
    -d = (-Complex.1) * d
    (-Complex.1) * d = -d
    b * (Complex.i * (Complex.i * d)) = b * (-d)
    neg_mul_right(b, d)
    b * (-d) = -(b * d)
    (Complex.i * b) * (Complex.i * d) = -(b * d)
    (a + Complex.i * b) * (c + Complex.i * d) = a * c + Complex.i * (a * d) + Complex.i * (b * c) + (-(b * d))
    add_assoc(a * c, Complex.i * (a * d), Complex.i * (b * c))
    (a * c + Complex.i * (a * d)) + Complex.i * (b * c) = a * c + (Complex.i * (a * d) + Complex.i * (b * c))
    a * c + Complex.i * (a * d) + Complex.i * (b * c) + (-(b * d)) = a * c + (Complex.i * (a * d) + Complex.i * (b * c)) + (-(b * d))
    distrib(Complex.i, a * d, b * c)
    Complex.i * (a * d) + Complex.i * (b * c) = Complex.i * (a * d + b * c)
    a * c + (Complex.i * (a * d) + Complex.i * (b * c)) + (-(b * d)) = a * c + Complex.i * (a * d + b * c) + (-(b * d))
    add_assoc(a * c, Complex.i * (a * d + b * c), -(b * d))
    a * c + Complex.i * (a * d + b * c) + (-(b * d)) = (a * c + Complex.i * (a * d + b * c)) + (-(b * d))
    add_assoc(a * c, Complex.i * (a * d + b * c), -(b * d))
    (a * c + Complex.i * (a * d + b * c)) + (-(b * d)) = a * c + (Complex.i * (a * d + b * c) + (-(b * d)))
    add_comm(Complex.i * (a * d + b * c), -(b * d))
    Complex.i * (a * d + b * c) + (-(b * d)) = -(b * d) + Complex.i * (a * d + b * c)
    a * c + (Complex.i * (a * d + b * c) + (-(b * d))) = a * c + (-(b * d) + Complex.i * (a * d + b * c))
    add_assoc(a * c, -(b * d), Complex.i * (a * d + b * c))
    a * c + (-(b * d) + Complex.i * (a * d + b * c)) = (a * c + -(b * d)) + Complex.i * (a * d + b * c)
    (a + Complex.i * b) * (c + Complex.i * d) = (a * c + -(b * d)) + Complex.i * (a * d + b * c)
    (a + Complex.i * b) * (c + Complex.i * d) = (a * c - b * d) + Complex.i * (a * d + b * c)
}

/// The expansion (a - i·b)·(c - i·d) = (a·c - b·d) - i·(a·d + b·c).
theorem complex_mul_neg_i_forms(a: Complex, b: Complex, c: Complex, d: Complex) {
    (a - Complex.i * b) * (c - Complex.i * d) = (a * c - b * d) - Complex.i * (a * d + b * c)
} by {
    distrib(a, -(Complex.i * b), c - Complex.i * d)
    (a - Complex.i * b) * (c - Complex.i * d) = a * (c - Complex.i * d) + (-(Complex.i * b)) * (c - Complex.i * d)
    mul_sub_left(a, c, Complex.i * d)
    a * (c - Complex.i * d) = a * c - a * (Complex.i * d)
    mul_comm(a, Complex.i * d)
    a * (Complex.i * d) = (Complex.i * d) * a
    mul_assoc(Complex.i, d, a)
    (Complex.i * d) * a = Complex.i * (d * a)
    mul_comm(d, a)
    d * a = a * d
    a * (Complex.i * d) = Complex.i * (a * d)
    a * (c - Complex.i * d) = a * c - Complex.i * (a * d)
    mul_sub_left(-(Complex.i * b), c, Complex.i * d)
    (-(Complex.i * b)) * (c - Complex.i * d) = (-(Complex.i * b)) * c - (-(Complex.i * b)) * (Complex.i * d)
    mul_neg_left[Complex](Complex.i * b, c)
    (-(Complex.i * b)) * c = -((Complex.i * b) * c)
    mul_assoc(Complex.i, b, c)
    (Complex.i * b) * c = Complex.i * (b * c)
    (-(Complex.i * b)) * c = -(Complex.i * (b * c))
    mul_neg_left[Complex](Complex.i * b, Complex.i * d)
    (-(Complex.i * b)) * (Complex.i * d) = -((Complex.i * b) * (Complex.i * d))
    mul_comm(Complex.i, b)
    Complex.i * b = b * Complex.i
    (Complex.i * b) * (Complex.i * d) = (b * Complex.i) * (Complex.i * d)
    mul_assoc(b, Complex.i, Complex.i * d)
    (b * Complex.i) * (Complex.i * d) = b * (Complex.i * (Complex.i * d))
    mul_assoc(Complex.i, Complex.i, d)
    Complex.i * (Complex.i * d) = (Complex.i * Complex.i) * d
    i_squared_eq_neg_one
    Complex.i * Complex.i = -Complex.1
    (Complex.i * Complex.i) * d = (-Complex.1) * d
    neg_eq_mul_neg_one(d)
    -d = (-Complex.1) * d
    (-Complex.1) * d = -d
    b * (Complex.i * (Complex.i * d)) = b * (-d)
    neg_mul_right(b, d)
    b * (-d) = -(b * d)
    (Complex.i * b) * (Complex.i * d) = -(b * d)
    (-(Complex.i * b)) * (Complex.i * d) = -(-(b * d))
    neg_neg_complex(b * d)
    -(-(b * d)) = b * d
    (-(Complex.i * b)) * (Complex.i * d) = b * d
    (-(Complex.i * b)) * (c - Complex.i * d) = -(Complex.i * (b * c)) - b * d
    (a - Complex.i * b) * (c - Complex.i * d) = a * c - Complex.i * (a * d) + (-(Complex.i * (b * c)) - b * d)
    add_assoc(a * c, -(Complex.i * (a * d)), -(Complex.i * (b * c)) + -(b * d))
    a * c - Complex.i * (a * d) + (-(Complex.i * (b * c)) - b * d) = a * c + (-(Complex.i * (a * d)) + (-(Complex.i * (b * c)) + -(b * d)))
    add_assoc(-(Complex.i * (a * d)), -(Complex.i * (b * c)), -(b * d))
    -(Complex.i * (a * d)) + (-(Complex.i * (b * c)) + -(b * d)) = (-(Complex.i * (a * d)) + -(Complex.i * (b * c))) + -(b * d)
    neg_add(Complex.i * (a * d), Complex.i * (b * c))
    -(Complex.i * (a * d)) + -(Complex.i * (b * c)) = -(Complex.i * (a * d) + Complex.i * (b * c))
    (-(Complex.i * (a * d)) + -(Complex.i * (b * c))) + -(b * d) = -(Complex.i * (a * d) + Complex.i * (b * c)) + -(b * d)
    a * c + (-(Complex.i * (a * d)) + (-(Complex.i * (b * c)) + -(b * d))) = a * c + (-(Complex.i * (a * d) + Complex.i * (b * c)) + -(b * d))
    add_assoc(a * c, -(Complex.i * (a * d) + Complex.i * (b * c)), -(b * d))
    a * c + (-(Complex.i * (a * d) + Complex.i * (b * c)) + -(b * d)) = (a * c + -(Complex.i * (a * d) + Complex.i * (b * c))) + -(b * d)
    a * c - Complex.i * (a * d) + (-(Complex.i * (b * c)) - b * d) = a * c - (Complex.i * (a * d) + Complex.i * (b * c)) - b * d
    distrib(Complex.i, a * d, b * c)
    Complex.i * (a * d) + Complex.i * (b * c) = Complex.i * (a * d + b * c)
    a * c - (Complex.i * (a * d) + Complex.i * (b * c)) - b * d = a * c - Complex.i * (a * d + b * c) - b * d
    a * c - Complex.i * (a * d + b * c) - b * d = (a * c + -(Complex.i * (a * d + b * c))) + -(b * d)
    add_assoc(a * c, -(Complex.i * (a * d + b * c)), -(b * d))
    (a * c + -(Complex.i * (a * d + b * c))) + -(b * d) = a * c + (-(Complex.i * (a * d + b * c)) + -(b * d))
    add_comm(-(Complex.i * (a * d + b * c)), -(b * d))
    -(Complex.i * (a * d + b * c)) + -(b * d) = -(b * d) + -(Complex.i * (a * d + b * c))
    a * c + (-(Complex.i * (a * d + b * c)) + -(b * d)) = a * c + (-(b * d) + -(Complex.i * (a * d + b * c)))
    add_assoc(a * c, -(b * d), -(Complex.i * (a * d + b * c)))
    a * c + (-(b * d) + -(Complex.i * (a * d + b * c))) = (a * c + -(b * d)) + -(Complex.i * (a * d + b * c))
    a * c - Complex.i * (a * d + b * c) - b * d = (a * c - b * d) - Complex.i * (a * d + b * c)
    (a - Complex.i * b) * (c - Complex.i * d) = (a * c - b * d) - Complex.i * (a * d + b * c)
}

// ---------------------------------------------------------------------------
// a. Parity.
// ---------------------------------------------------------------------------

/// Cosine is even: (-z).cos = z.cos.
theorem complex_cos_neg(z: Complex) {
    complex_cos(-z) = complex_cos(z)
} by {
    complex_cos(-z) = (complex_exp(Complex.i * (-z)) + complex_exp(-(Complex.i * (-z)))) / Complex.from_real(two)
    neg_mul_right(Complex.i, z)
    Complex.i * (-z) = -(Complex.i * z)
    complex_exp(Complex.i * (-z)) = complex_exp(-(Complex.i * z))
    Complex.i * (-z) = -(Complex.i * z)
    -(Complex.i * (-z)) = -(-(Complex.i * z))
    neg_neg_complex(Complex.i * z)
    -(-(Complex.i * z)) = Complex.i * z
    -(Complex.i * (-z)) = Complex.i * z
    complex_exp(-(Complex.i * (-z))) = complex_exp(Complex.i * z)
    complex_cos(-z) = (complex_exp(-(Complex.i * z)) + complex_exp(Complex.i * z)) / Complex.from_real(two)
    add_comm(complex_exp(Complex.i * z), complex_exp(-(Complex.i * z)))
    complex_exp(Complex.i * z) + complex_exp(-(Complex.i * z)) = complex_exp(-(Complex.i * z)) + complex_exp(Complex.i * z)
    complex_cos(-z) = (complex_exp(Complex.i * z) + complex_exp(-(Complex.i * z))) / Complex.from_real(two)
    complex_cos(z) = (complex_exp(Complex.i * z) + complex_exp(-(Complex.i * z))) / Complex.from_real(two)
    complex_cos(-z) = complex_cos(z)
}

/// Sine is odd: (-z).sin = -z.sin.
theorem complex_sin_neg(z: Complex) {
    complex_sin(-z) = -complex_sin(z)
} by {
    complex_sin(-z) = (complex_exp(Complex.i * (-z)) - complex_exp(-(Complex.i * (-z)))) / (Complex.from_real(two) * Complex.i)
    neg_mul_right(Complex.i, z)
    Complex.i * (-z) = -(Complex.i * z)
    complex_exp(Complex.i * (-z)) = complex_exp(-(Complex.i * z))
    Complex.i * (-z) = -(Complex.i * z)
    -(Complex.i * (-z)) = -(-(Complex.i * z))
    neg_neg_complex(Complex.i * z)
    -(-(Complex.i * z)) = Complex.i * z
    -(Complex.i * (-z)) = Complex.i * z
    complex_exp(-(Complex.i * (-z))) = complex_exp(Complex.i * z)
    complex_sin(-z) = (complex_exp(-(Complex.i * z)) - complex_exp(Complex.i * z)) / (Complex.from_real(two) * Complex.i)
    neg_sub(complex_exp(Complex.i * z), complex_exp(-(Complex.i * z)))
    -(complex_exp(Complex.i * z) - complex_exp(-(Complex.i * z))) = complex_exp(-(Complex.i * z)) - complex_exp(Complex.i * z)
    complex_sin(-z) = -(complex_exp(Complex.i * z) - complex_exp(-(Complex.i * z))) / (Complex.from_real(two) * Complex.i)
    neg_div_num(complex_exp(Complex.i * z) - complex_exp(-(Complex.i * z)), Complex.from_real(two) * Complex.i)
    -(complex_exp(Complex.i * z) - complex_exp(-(Complex.i * z))) / (Complex.from_real(two) * Complex.i) = -((complex_exp(Complex.i * z) - complex_exp(-(Complex.i * z))) / (Complex.from_real(two) * Complex.i))
    complex_sin(-z) = -((complex_exp(Complex.i * z) - complex_exp(-(Complex.i * z))) / (Complex.from_real(two) * Complex.i))
    complex_sin(z) = (complex_exp(Complex.i * z) - complex_exp(-(Complex.i * z))) / (Complex.from_real(two) * Complex.i)
    complex_sin(-z) = -complex_sin(z)
}

// ---------------------------------------------------------------------------
// b. Euler's formula for complex arguments.
// ---------------------------------------------------------------------------

/// Euler's formula for complex arguments: exp(i·z) = z.cos + i·z.sin.
theorem complex_euler_complex(z: Complex) {
    complex_exp(Complex.i * z) = complex_cos(z) + Complex.i * complex_sin(z)
} by {
    complex_cos(z) = (complex_exp(Complex.i * z) + complex_exp(-(Complex.i * z))) / Complex.from_real(two)
    complex_sin(z) = (complex_exp(Complex.i * z) - complex_exp(-(Complex.i * z))) / (Complex.from_real(two) * Complex.i)
    Complex.i * complex_sin(z) = Complex.i * ((complex_exp(Complex.i * z) - complex_exp(-(Complex.i * z))) / (Complex.from_real(two) * Complex.i))
    (complex_exp(Complex.i * z) - complex_exp(-(Complex.i * z))) / (Complex.from_real(two) * Complex.i) = (complex_exp(Complex.i * z) - complex_exp(-(Complex.i * z))) * (Complex.from_real(two) * Complex.i).inverse
    Complex.i * ((complex_exp(Complex.i * z) - complex_exp(-(Complex.i * z))) * (Complex.from_real(two) * Complex.i).inverse) = (Complex.i * (complex_exp(Complex.i * z) - complex_exp(-(Complex.i * z)))) * (Complex.from_real(two) * Complex.i).inverse
    Complex.i * (complex_exp(Complex.i * z) - complex_exp(-(Complex.i * z))) = (complex_exp(Complex.i * z) - complex_exp(-(Complex.i * z))) * Complex.i
    (Complex.i * (complex_exp(Complex.i * z) - complex_exp(-(Complex.i * z)))) * (Complex.from_real(two) * Complex.i).inverse = ((complex_exp(Complex.i * z) - complex_exp(-(Complex.i * z))) * Complex.i) * (Complex.from_real(two) * Complex.i).inverse
    mul_assoc(complex_exp(Complex.i * z) - complex_exp(-(Complex.i * z)), Complex.i, (Complex.from_real(two) * Complex.i).inverse)
    ((complex_exp(Complex.i * z) - complex_exp(-(Complex.i * z))) * Complex.i) * (Complex.from_real(two) * Complex.i).inverse = (complex_exp(Complex.i * z) - complex_exp(-(Complex.i * z))) * (Complex.i * (Complex.from_real(two) * Complex.i).inverse)
    complex_i_over_two_i
    Complex.i * (Complex.from_real(two) * Complex.i).inverse = Complex.from_real(two).inverse
    Complex.i * complex_sin(z) = (complex_exp(Complex.i * z) - complex_exp(-(Complex.i * z))) * Complex.from_real(two).inverse
    (complex_exp(Complex.i * z) - complex_exp(-(Complex.i * z))) * Complex.from_real(two).inverse = (complex_exp(Complex.i * z) - complex_exp(-(Complex.i * z))) / Complex.from_real(two)
    Complex.i * complex_sin(z) = (complex_exp(Complex.i * z) - complex_exp(-(Complex.i * z))) / Complex.from_real(two)
    complex_cos(z) + Complex.i * complex_sin(z) = (complex_exp(Complex.i * z) + complex_exp(-(Complex.i * z))) / Complex.from_real(two) +
        (complex_exp(Complex.i * z) - complex_exp(-(Complex.i * z))) / Complex.from_real(two)
    div_distrib_right(complex_exp(Complex.i * z) + complex_exp(-(Complex.i * z)),
        complex_exp(Complex.i * z) - complex_exp(-(Complex.i * z)), Complex.from_real(two))
    (complex_exp(Complex.i * z) + complex_exp(-(Complex.i * z))) / Complex.from_real(two) +
        (complex_exp(Complex.i * z) - complex_exp(-(Complex.i * z))) / Complex.from_real(two) = ((complex_exp(Complex.i * z) + complex_exp(-(Complex.i * z))) +
            (complex_exp(Complex.i * z) - complex_exp(-(Complex.i * z)))) / Complex.from_real(two)
    complex_add_sub_same(complex_exp(Complex.i * z), complex_exp(-(Complex.i * z)))
    (complex_exp(Complex.i * z) + complex_exp(-(Complex.i * z))) +
        (complex_exp(Complex.i * z) - complex_exp(-(Complex.i * z))) = Complex.from_real(two) * complex_exp(Complex.i * z)
    complex_cos(z) + Complex.i * complex_sin(z) = (Complex.from_real(two) * complex_exp(Complex.i * z)) / Complex.from_real(two)
    complex_from_real_two_nonzero
    Complex.from_real(two) != Complex.0
    complex_div_mul_cancel(Complex.from_real(two), complex_exp(Complex.i * z))
    Complex.from_real(two) != Complex.0 implies (Complex.from_real(two) * complex_exp(Complex.i * z)) / Complex.from_real(two) = complex_exp(Complex.i * z)
    (Complex.from_real(two) * complex_exp(Complex.i * z)) / Complex.from_real(two) = complex_exp(Complex.i * z)
    complex_cos(z) + Complex.i * complex_sin(z) = complex_exp(Complex.i * z)
    complex_exp(Complex.i * z) = complex_cos(z) + Complex.i * complex_sin(z)
}

/// Euler's formula with a negative argument: exp(-i·z) = z.cos - i·z.sin.
theorem complex_exp_neg_i_mul(z: Complex) {
    complex_exp(-(Complex.i * z)) = complex_cos(z) - Complex.i * complex_sin(z)
} by {
    neg_mul_right(Complex.i, z)
    Complex.i * (-z) = -(Complex.i * z)
    complex_exp(Complex.i * (-z)) = complex_exp(-(Complex.i * z))
    complex_euler_complex(-z)
    complex_exp(Complex.i * (-z)) = complex_cos(-z) + Complex.i * complex_sin(-z)
    complex_cos_neg(z)
    complex_cos(-z) = complex_cos(z)
    complex_sin_neg(z)
    complex_sin(-z) = -complex_sin(z)
    Complex.i * complex_sin(-z) = Complex.i * (-complex_sin(z))
    neg_mul_right(Complex.i, complex_sin(z))
    Complex.i * (-complex_sin(z)) = -(Complex.i * complex_sin(z))
    complex_cos(-z) + Complex.i * complex_sin(-z) = complex_cos(z) - Complex.i * complex_sin(z)
    complex_exp(-(Complex.i * z)) = complex_cos(z) - Complex.i * complex_sin(z)
}

// ---------------------------------------------------------------------------
// c. The Pythagorean identity.
// ---------------------------------------------------------------------------

/// The fundamental Pythagorean identity: (z).cos² + (z).sin² = 1.
theorem complex_cos_sq_add_sin_sq(z: Complex) {
    complex_cos(z) * complex_cos(z) + complex_sin(z) * complex_sin(z) = Complex.1
} by {
    complex_euler_complex(z)
    complex_exp(Complex.i * z) = complex_cos(z) + Complex.i * complex_sin(z)
    complex_exp_neg_i_mul(z)
    complex_exp(-(Complex.i * z)) = complex_cos(z) - Complex.i * complex_sin(z)
    complex_exp_mul_neg(Complex.i * z)
    complex_exp(Complex.i * z) * complex_exp(-(Complex.i * z)) = Complex.1
    mul_conj_expansion(complex_cos(z), complex_sin(z))
    (complex_cos(z) + Complex.i * complex_sin(z)) * (complex_cos(z) - Complex.i * complex_sin(z)) = complex_cos(z) * complex_cos(z) + complex_sin(z) * complex_sin(z)
    complex_exp(Complex.i * z) * complex_exp(-(Complex.i * z)) = (complex_cos(z) + Complex.i * complex_sin(z)) * (complex_cos(z) - Complex.i * complex_sin(z))
    complex_cos(z) * complex_cos(z) + complex_sin(z) * complex_sin(z) = Complex.1
}

// ---------------------------------------------------------------------------
// d. The addition formulas.
// ---------------------------------------------------------------------------

/// The sine addition formula: (z + w).sin = z.sin·w.cos + z.cos·w.sin.
theorem complex_sin_add(z: Complex, w: Complex) {
    complex_sin(z + w) = complex_sin(z) * complex_cos(w) + complex_cos(z) * complex_sin(w)
} by {
    complex_sin(z + w) = (complex_exp(Complex.i * (z + w)) - complex_exp(-(Complex.i * (z + w)))) / (Complex.from_real(two) * Complex.i)
    distrib(Complex.i, z, w)
    Complex.i * (z + w) = Complex.i * z + Complex.i * w
    complex_exp(Complex.i * (z + w)) = complex_exp(Complex.i * z + Complex.i * w)
    complex_exp_add(Complex.i * z, Complex.i * w)
    complex_exp(Complex.i * z + Complex.i * w) = complex_exp(Complex.i * z) * complex_exp(Complex.i * w)
    complex_exp(Complex.i * (z + w)) = complex_exp(Complex.i * z) * complex_exp(Complex.i * w)
    neg_add(Complex.i * z, Complex.i * w)
    -(Complex.i * z + Complex.i * w) = -(Complex.i * z) + -(Complex.i * w)
    complex_exp(-(Complex.i * (z + w))) = complex_exp(-(Complex.i * z) + -(Complex.i * w))
    complex_exp_add(-(Complex.i * z), -(Complex.i * w))
    complex_exp(-(Complex.i * z) + -(Complex.i * w)) = complex_exp(-(Complex.i * z)) * complex_exp(-(Complex.i * w))
    complex_exp(-(Complex.i * (z + w))) = complex_exp(-(Complex.i * z)) * complex_exp(-(Complex.i * w))
    complex_euler_complex(z)
    complex_exp(Complex.i * z) = complex_cos(z) + Complex.i * complex_sin(z)
    complex_euler_complex(w)
    complex_exp(Complex.i * w) = complex_cos(w) + Complex.i * complex_sin(w)
    complex_exp_neg_i_mul(z)
    complex_exp(-(Complex.i * z)) = complex_cos(z) - Complex.i * complex_sin(z)
    complex_exp_neg_i_mul(w)
    complex_exp(-(Complex.i * w)) = complex_cos(w) - Complex.i * complex_sin(w)
    complex_exp(Complex.i * (z + w)) = (complex_cos(z) + Complex.i * complex_sin(z)) * (complex_cos(w) + Complex.i * complex_sin(w))
    complex_exp(-(Complex.i * (z + w))) = (complex_cos(z) - Complex.i * complex_sin(z)) * (complex_cos(w) - Complex.i * complex_sin(w))
    complex_mul_i_forms(complex_cos(z), complex_sin(z), complex_cos(w), complex_sin(w))
    (complex_cos(z) + Complex.i * complex_sin(z)) * (complex_cos(w) + Complex.i * complex_sin(w)) = (complex_cos(z) * complex_cos(w) - complex_sin(z) * complex_sin(w)) +
        Complex.i * (complex_cos(z) * complex_sin(w) + complex_sin(z) * complex_cos(w))
    complex_mul_neg_i_forms(complex_cos(z), complex_sin(z), complex_cos(w), complex_sin(w))
    (complex_cos(z) - Complex.i * complex_sin(z)) * (complex_cos(w) - Complex.i * complex_sin(w)) = (complex_cos(z) * complex_cos(w) - complex_sin(z) * complex_sin(w)) -
        Complex.i * (complex_cos(z) * complex_sin(w) + complex_sin(z) * complex_cos(w))
    complex_exp(Complex.i * (z + w)) = (complex_cos(z) * complex_cos(w) - complex_sin(z) * complex_sin(w)) +
        Complex.i * (complex_cos(z) * complex_sin(w) + complex_sin(z) * complex_cos(w))
    complex_exp(-(Complex.i * (z + w))) = (complex_cos(z) * complex_cos(w) - complex_sin(z) * complex_sin(w)) -
        Complex.i * (complex_cos(z) * complex_sin(w) + complex_sin(z) * complex_cos(w))
    complex_add_sub_cancel_twice(complex_cos(z) * complex_cos(w) - complex_sin(z) * complex_sin(w),
        Complex.i * (complex_cos(z) * complex_sin(w) + complex_sin(z) * complex_cos(w)))
    ((complex_cos(z) * complex_cos(w) - complex_sin(z) * complex_sin(w)) +
        Complex.i * (complex_cos(z) * complex_sin(w) + complex_sin(z) * complex_cos(w))) -
        ((complex_cos(z) * complex_cos(w) - complex_sin(z) * complex_sin(w)) -
            Complex.i * (complex_cos(z) * complex_sin(w) + complex_sin(z) * complex_cos(w))) = Complex.from_real(two) * (Complex.i * (complex_cos(z) * complex_sin(w) + complex_sin(z) * complex_cos(w)))
    complex_sin(z + w) = (Complex.from_real(two) * (Complex.i * (complex_cos(z) * complex_sin(w) + complex_sin(z) * complex_cos(w)))) /
        (Complex.from_real(two) * Complex.i)
    mul_assoc(Complex.from_real(two), Complex.i, complex_cos(z) * complex_sin(w) + complex_sin(z) * complex_cos(w))
    (Complex.from_real(two) * Complex.i) * (complex_cos(z) * complex_sin(w) + complex_sin(z) * complex_cos(w)) = Complex.from_real(two) * (Complex.i * (complex_cos(z) * complex_sin(w) + complex_sin(z) * complex_cos(w)))
    complex_sin(z + w) = ((Complex.from_real(two) * Complex.i) * (complex_cos(z) * complex_sin(w) + complex_sin(z) * complex_cos(w))) /
        (Complex.from_real(two) * Complex.i)
    complex_two_i_nonzero
    Complex.from_real(two) * Complex.i != Complex.0
    complex_div_mul_cancel(Complex.from_real(two) * Complex.i,
        complex_cos(z) * complex_sin(w) + complex_sin(z) * complex_cos(w))
    Complex.from_real(two) * Complex.i != Complex.0 implies ((Complex.from_real(two) * Complex.i) * (complex_cos(z) * complex_sin(w) + complex_sin(z) * complex_cos(w))) /
        (Complex.from_real(two) * Complex.i) = complex_cos(z) * complex_sin(w) + complex_sin(z) * complex_cos(w)
    complex_sin(z + w) = complex_cos(z) * complex_sin(w) + complex_sin(z) * complex_cos(w)
    mul_comm(complex_cos(z), complex_sin(w))
    complex_cos(z) * complex_sin(w) = complex_sin(w) * complex_cos(z)
    complex_sin(z + w) = complex_sin(z) * complex_cos(w) + complex_cos(z) * complex_sin(w)
}

/// The cosine addition formula: (z + w).cos = z.cos·w.cos - z.sin·w.sin.
theorem complex_cos_add(z: Complex, w: Complex) {
    complex_cos(z + w) = complex_cos(z) * complex_cos(w) - complex_sin(z) * complex_sin(w)
} by {
    complex_cos(z + w) = (complex_exp(Complex.i * (z + w)) + complex_exp(-(Complex.i * (z + w)))) / Complex.from_real(two)
    distrib(Complex.i, z, w)
    Complex.i * (z + w) = Complex.i * z + Complex.i * w
    complex_exp(Complex.i * (z + w)) = complex_exp(Complex.i * z + Complex.i * w)
    complex_exp_add(Complex.i * z, Complex.i * w)
    complex_exp(Complex.i * z + Complex.i * w) = complex_exp(Complex.i * z) * complex_exp(Complex.i * w)
    complex_exp(Complex.i * (z + w)) = complex_exp(Complex.i * z) * complex_exp(Complex.i * w)
    neg_add(Complex.i * z, Complex.i * w)
    -(Complex.i * z + Complex.i * w) = -(Complex.i * z) + -(Complex.i * w)
    complex_exp(-(Complex.i * (z + w))) = complex_exp(-(Complex.i * z) + -(Complex.i * w))
    complex_exp_add(-(Complex.i * z), -(Complex.i * w))
    complex_exp(-(Complex.i * z) + -(Complex.i * w)) = complex_exp(-(Complex.i * z)) * complex_exp(-(Complex.i * w))
    complex_exp(-(Complex.i * (z + w))) = complex_exp(-(Complex.i * z)) * complex_exp(-(Complex.i * w))
    complex_euler_complex(z)
    complex_exp(Complex.i * z) = complex_cos(z) + Complex.i * complex_sin(z)
    complex_euler_complex(w)
    complex_exp(Complex.i * w) = complex_cos(w) + Complex.i * complex_sin(w)
    complex_exp_neg_i_mul(z)
    complex_exp(-(Complex.i * z)) = complex_cos(z) - Complex.i * complex_sin(z)
    complex_exp_neg_i_mul(w)
    complex_exp(-(Complex.i * w)) = complex_cos(w) - Complex.i * complex_sin(w)
    complex_exp(Complex.i * (z + w)) = (complex_cos(z) + Complex.i * complex_sin(z)) * (complex_cos(w) + Complex.i * complex_sin(w))
    complex_exp(-(Complex.i * (z + w))) = (complex_cos(z) - Complex.i * complex_sin(z)) * (complex_cos(w) - Complex.i * complex_sin(w))
    complex_mul_i_forms(complex_cos(z), complex_sin(z), complex_cos(w), complex_sin(w))
    (complex_cos(z) + Complex.i * complex_sin(z)) * (complex_cos(w) + Complex.i * complex_sin(w)) = (complex_cos(z) * complex_cos(w) - complex_sin(z) * complex_sin(w)) +
        Complex.i * (complex_cos(z) * complex_sin(w) + complex_sin(z) * complex_cos(w))
    complex_mul_neg_i_forms(complex_cos(z), complex_sin(z), complex_cos(w), complex_sin(w))
    (complex_cos(z) - Complex.i * complex_sin(z)) * (complex_cos(w) - Complex.i * complex_sin(w)) = (complex_cos(z) * complex_cos(w) - complex_sin(z) * complex_sin(w)) -
        Complex.i * (complex_cos(z) * complex_sin(w) + complex_sin(z) * complex_cos(w))
    complex_exp(Complex.i * (z + w)) = (complex_cos(z) * complex_cos(w) - complex_sin(z) * complex_sin(w)) +
        Complex.i * (complex_cos(z) * complex_sin(w) + complex_sin(z) * complex_cos(w))
    complex_exp(-(Complex.i * (z + w))) = (complex_cos(z) * complex_cos(w) - complex_sin(z) * complex_sin(w)) -
        Complex.i * (complex_cos(z) * complex_sin(w) + complex_sin(z) * complex_cos(w))
    complex_add_sub_same(complex_cos(z) * complex_cos(w) - complex_sin(z) * complex_sin(w),
        Complex.i * (complex_cos(z) * complex_sin(w) + complex_sin(z) * complex_cos(w)))
    ((complex_cos(z) * complex_cos(w) - complex_sin(z) * complex_sin(w)) +
        Complex.i * (complex_cos(z) * complex_sin(w) + complex_sin(z) * complex_cos(w))) +
        ((complex_cos(z) * complex_cos(w) - complex_sin(z) * complex_sin(w)) -
            Complex.i * (complex_cos(z) * complex_sin(w) + complex_sin(z) * complex_cos(w))) = Complex.from_real(two) * (complex_cos(z) * complex_cos(w) - complex_sin(z) * complex_sin(w))
    complex_cos(z + w) = (Complex.from_real(two) * (complex_cos(z) * complex_cos(w) - complex_sin(z) * complex_sin(w))) /
        Complex.from_real(two)
    complex_from_real_two_nonzero
    Complex.from_real(two) != Complex.0
    complex_div_mul_cancel(Complex.from_real(two), complex_cos(z) * complex_cos(w) - complex_sin(z) * complex_sin(w))
    Complex.from_real(two) != Complex.0 implies (Complex.from_real(two) * (complex_cos(z) * complex_cos(w) - complex_sin(z) * complex_sin(w))) /
        Complex.from_real(two) = complex_cos(z) * complex_cos(w) - complex_sin(z) * complex_sin(w)
    complex_cos(z + w) = complex_cos(z) * complex_cos(w) - complex_sin(z) * complex_sin(w)
}

/// The sine double-angle formula: (2·z).sin = 2·z.sin·z.cos.
theorem complex_sin_double(z: Complex) {
    complex_sin(Complex.from_real(two) * z) = Complex.from_real(two) * (complex_sin(z) * complex_cos(z))
} by {
    complex_two_mul(z)
    Complex.from_real(two) * z = z + z
    complex_sin(Complex.from_real(two) * z) = complex_sin(z + z)
    complex_sin_add(z, z)
    complex_sin(z + z) = complex_sin(z) * complex_cos(z) + complex_cos(z) * complex_sin(z)
    mul_comm(complex_cos(z), complex_sin(z))
    complex_cos(z) * complex_sin(z) = complex_sin(z) * complex_cos(z)
    complex_sin(z) * complex_cos(z) + complex_cos(z) * complex_sin(z) = complex_sin(z) * complex_cos(z) + complex_sin(z) * complex_cos(z)
    complex_two_mul(complex_sin(z) * complex_cos(z))
    Complex.from_real(two) * (complex_sin(z) * complex_cos(z)) = complex_sin(z) * complex_cos(z) + complex_sin(z) * complex_cos(z)
    complex_sin(z + z) = Complex.from_real(two) * (complex_sin(z) * complex_cos(z))
    complex_sin(Complex.from_real(two) * z) = Complex.from_real(two) * (complex_sin(z) * complex_cos(z))
}

/// The cosine double-angle formula: (2·z).cos = (z).cos² - (z).sin².
theorem complex_cos_double(z: Complex) {
    complex_cos(Complex.from_real(two) * z) = complex_cos(z) * complex_cos(z) - complex_sin(z) * complex_sin(z)
} by {
    complex_two_mul(z)
    Complex.from_real(two) * z = z + z
    complex_cos(Complex.from_real(two) * z) = complex_cos(z + z)
    complex_cos_add(z, z)
    complex_cos(z + z) = complex_cos(z) * complex_cos(z) - complex_sin(z) * complex_sin(z)
    complex_cos(Complex.from_real(two) * z) = complex_cos(z) * complex_cos(z) - complex_sin(z) * complex_sin(z)
}

// ---------------------------------------------------------------------------
// e. Periodicity.
// ---------------------------------------------------------------------------

/// The complex sine has period 2·pi: (z + 2·pi).sin = z.sin.
theorem complex_sin_periodic(z: Complex) {
    complex_sin(z + Complex.from_real(two * pi)) = complex_sin(z)
} by {
    complex_sin(z + Complex.from_real(two * pi)) = (complex_exp(Complex.i * (z + Complex.from_real(two * pi))) -
            complex_exp(-(Complex.i * (z + Complex.from_real(two * pi))))) /
        (Complex.from_real(two) * Complex.i)
    distrib(Complex.i, z, Complex.from_real(two * pi))
    Complex.i * (z + Complex.from_real(two * pi)) = Complex.i * z + Complex.i * Complex.from_real(two * pi)
    complex_i_mul_real(two * pi) = Complex.i * Complex.from_real(two * pi)
    Complex.i * (z + Complex.from_real(two * pi)) = Complex.i * z + complex_i_mul_real(two * pi)
    complex_exp(Complex.i * (z + Complex.from_real(two * pi))) = complex_exp(Complex.i * z + complex_i_mul_real(two * pi))
    complex_exp_add(Complex.i * z, complex_i_mul_real(two * pi))
    complex_exp(Complex.i * z + complex_i_mul_real(two * pi)) = complex_exp(Complex.i * z) * complex_exp(complex_i_mul_real(two * pi))
    complex_exp_two_pi
    complex_exp(complex_i_mul_real(two * pi)) = Complex.1
    complex_exp(Complex.i * (z + Complex.from_real(two * pi))) = complex_exp(Complex.i * z) * Complex.1
    mul_one_right(complex_exp(Complex.i * z))
    complex_exp(Complex.i * z) * Complex.1 = complex_exp(Complex.i * z)
    complex_exp(Complex.i * (z + Complex.from_real(two * pi))) = complex_exp(Complex.i * z)
    neg_add(Complex.i * z, complex_i_mul_real(two * pi))
    -(Complex.i * z + complex_i_mul_real(two * pi)) = -(Complex.i * z) + -(complex_i_mul_real(two * pi))
    complex_exp(-(Complex.i * (z + Complex.from_real(two * pi)))) = complex_exp(-(Complex.i * z) + -(complex_i_mul_real(two * pi)))
    complex_exp_add(-(Complex.i * z), -(complex_i_mul_real(two * pi)))
    complex_exp(-(Complex.i * z) + -(complex_i_mul_real(two * pi))) = complex_exp(-(Complex.i * z)) * complex_exp(-(complex_i_mul_real(two * pi)))
    complex_exp_neg(complex_i_mul_real(two * pi))
    complex_exp(-(complex_i_mul_real(two * pi))) = complex_exp(complex_i_mul_real(two * pi)).inverse
    complex_exp_two_pi
    complex_exp(complex_i_mul_real(two * pi)) = Complex.1
    inverse_one_restated
    Complex.1.inverse = Complex.1
    complex_exp(-(complex_i_mul_real(two * pi))) = Complex.1
    complex_exp(-(Complex.i * (z + Complex.from_real(two * pi)))) = complex_exp(-(Complex.i * z)) * Complex.1
    mul_one_right(complex_exp(-(Complex.i * z)))
    complex_exp(-(Complex.i * z)) * Complex.1 = complex_exp(-(Complex.i * z))
    complex_exp(-(Complex.i * (z + Complex.from_real(two * pi)))) = complex_exp(-(Complex.i * z))
    complex_sin(z + Complex.from_real(two * pi)) = (complex_exp(Complex.i * z) - complex_exp(-(Complex.i * z))) / (Complex.from_real(two) * Complex.i)
    complex_sin(z) = (complex_exp(Complex.i * z) - complex_exp(-(Complex.i * z))) / (Complex.from_real(two) * Complex.i)
    complex_sin(z + Complex.from_real(two * pi)) = complex_sin(z)
}

/// The complex cosine has period 2·pi: (z + 2·pi).cos = z.cos.
theorem complex_cos_periodic(z: Complex) {
    complex_cos(z + Complex.from_real(two * pi)) = complex_cos(z)
} by {
    complex_cos(z + Complex.from_real(two * pi)) = (complex_exp(Complex.i * (z + Complex.from_real(two * pi))) +
            complex_exp(-(Complex.i * (z + Complex.from_real(two * pi))))) /
        Complex.from_real(two)
    distrib(Complex.i, z, Complex.from_real(two * pi))
    Complex.i * (z + Complex.from_real(two * pi)) = Complex.i * z + Complex.i * Complex.from_real(two * pi)
    complex_i_mul_real(two * pi) = Complex.i * Complex.from_real(two * pi)
    Complex.i * (z + Complex.from_real(two * pi)) = Complex.i * z + complex_i_mul_real(two * pi)
    complex_exp(Complex.i * (z + Complex.from_real(two * pi))) = complex_exp(Complex.i * z + complex_i_mul_real(two * pi))
    complex_exp_add(Complex.i * z, complex_i_mul_real(two * pi))
    complex_exp(Complex.i * z + complex_i_mul_real(two * pi)) = complex_exp(Complex.i * z) * complex_exp(complex_i_mul_real(two * pi))
    complex_exp_two_pi
    complex_exp(complex_i_mul_real(two * pi)) = Complex.1
    complex_exp(Complex.i * (z + Complex.from_real(two * pi))) = complex_exp(Complex.i * z) * Complex.1
    mul_one_right(complex_exp(Complex.i * z))
    complex_exp(Complex.i * z) * Complex.1 = complex_exp(Complex.i * z)
    complex_exp(Complex.i * (z + Complex.from_real(two * pi))) = complex_exp(Complex.i * z)
    neg_add(Complex.i * z, complex_i_mul_real(two * pi))
    -(Complex.i * z + complex_i_mul_real(two * pi)) = -(Complex.i * z) + -(complex_i_mul_real(two * pi))
    complex_exp(-(Complex.i * (z + Complex.from_real(two * pi)))) = complex_exp(-(Complex.i * z) + -(complex_i_mul_real(two * pi)))
    complex_exp_add(-(Complex.i * z), -(complex_i_mul_real(two * pi)))
    complex_exp(-(Complex.i * z) + -(complex_i_mul_real(two * pi))) = complex_exp(-(Complex.i * z)) * complex_exp(-(complex_i_mul_real(two * pi)))
    complex_exp_neg(complex_i_mul_real(two * pi))
    complex_exp(-(complex_i_mul_real(two * pi))) = complex_exp(complex_i_mul_real(two * pi)).inverse
    complex_exp_two_pi
    complex_exp(complex_i_mul_real(two * pi)) = Complex.1
    inverse_one_restated
    Complex.1.inverse = Complex.1
    complex_exp(-(complex_i_mul_real(two * pi))) = Complex.1
    complex_exp(-(Complex.i * (z + Complex.from_real(two * pi)))) = complex_exp(-(Complex.i * z)) * Complex.1
    mul_one_right(complex_exp(-(Complex.i * z)))
    complex_exp(-(Complex.i * z)) * Complex.1 = complex_exp(-(Complex.i * z))
    complex_exp(-(Complex.i * (z + Complex.from_real(two * pi)))) = complex_exp(-(Complex.i * z))
    complex_cos(z + Complex.from_real(two * pi)) = (complex_exp(Complex.i * z) + complex_exp(-(Complex.i * z))) / Complex.from_real(two)
    complex_cos(z) = (complex_exp(Complex.i * z) + complex_exp(-(Complex.i * z))) / Complex.from_real(two)
    complex_cos(z + Complex.from_real(two * pi)) = complex_cos(z)
}

// ---------------------------------------------------------------------------
// f. Special values.
// ---------------------------------------------------------------------------

/// The complex cosine of pi is negative one: pi.cos = -1.
theorem complex_cos_pi {
    complex_cos(Complex.from_real(pi)) = -Complex.1
} by {
    complex_cos_from_real(pi)
    complex_cos(Complex.from_real(pi)) = Complex.from_real(pi.cos)
    cos_pi_neg_one
    pi.cos = -Real.1
    Complex.from_real(pi.cos) = Complex.from_real(-Real.1)
    from_real_neg(Real.1)
    Complex.from_real(-Real.1) = -Complex.from_real(Real.1)
    from_real_one
    Complex.from_real(Real.1) = Complex.1
    Complex.from_real(-Real.1) = -Complex.1
    complex_cos(Complex.from_real(pi)) = -Complex.1
}

/// The complex sine of pi is zero: pi.sin = 0.
theorem complex_sin_pi {
    complex_sin(Complex.from_real(pi)) = Complex.0
} by {
    complex_sin_from_real(pi)
    complex_sin(Complex.from_real(pi)) = Complex.from_real(pi.sin)
    sin_pi_zero
    pi.sin = Real.0
    Complex.from_real(pi.sin) = Complex.from_real(Real.0)
    from_real_zero
    Complex.from_real(Real.0) = Complex.0
    complex_sin(Complex.from_real(pi)) = Complex.0
}

