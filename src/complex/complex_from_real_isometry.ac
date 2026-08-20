from nat import Nat
from real import Real, converges_to
from data.basic.functions import is_injective_fn, compose
from analysis import real_distance, real_converges_to_imp_tendsto_metric, is_isometry,
    isometry_is_injective_fn, is_lipschitz_with, is_lipschitz,
    isometry_is_lipschitz_with_one, isometry_is_lipschitz, tendsto_metric,
    is_cauchy_metric, isometry_preserves_tendsto_metric,
    isometry_preserves_cauchy_metric
from complex.complex import from_real_sub
from complex.complex_abs import modulus_from_real
from complex.complex_metric import Complex, complex_distance_eq_modulus_sub

numerals Real

/// The embedding of real numbers into the complex plane preserves pointwise metric distance.
theorem complex_from_real_distance(a: Real, b: Real) {
    Complex.from_real(a).distance(Complex.from_real(b)) = a.distance(b)
} by {
    complex_distance_eq_modulus_sub(Complex.from_real(a), Complex.from_real(b))
    from_real_sub(a, b)
    modulus_from_real(a - b)
    real_distance(a, b) = (a - b).abs
}

/// The real embedding into the complex plane is an isometry.
theorem complex_from_real_is_isometry {
    is_isometry[Real, Complex](Complex.from_real)
} by {
    forall(a: Real, b: Real) {
        complex_from_real_distance(a, b)
        Complex.from_real(a).distance(Complex.from_real(b)) = a.distance(b)
    }
}

/// The real embedding into the complex plane is injective.
theorem complex_from_real_is_injective_fn {
    is_injective_fn(Complex.from_real)
} by {
    complex_from_real_is_isometry
    isometry_is_injective_fn(Complex.from_real)
}

/// The real embedding into the complex plane is one-Lipschitz.
theorem complex_from_real_is_lipschitz_with_one {
    is_lipschitz_with(1, Complex.from_real)
} by {
    complex_from_real_is_isometry
    isometry_is_lipschitz_with_one(Complex.from_real)
}

/// The real embedding into the complex plane is Lipschitz.
theorem complex_from_real_is_lipschitz {
    is_lipschitz(Complex.from_real)
} by {
    complex_from_real_is_isometry
    isometry_is_lipschitz(Complex.from_real)
}

/// Metric convergence of a real sequence transports through the real embedding into `Complex`.
theorem complex_from_real_tendsto_metric(q: Nat -> Real, a: Real) {
    tendsto_metric(q, a) implies tendsto_metric(compose(Complex.from_real, q), Complex.from_real(a))
} by {
    if tendsto_metric(q, a) {
        complex_from_real_is_isometry
        isometry_preserves_tendsto_metric(Complex.from_real, q, a)
        tendsto_metric(compose(Complex.from_real, q), Complex.from_real(a))
    }
}

/// Metric-Cauchy real sequences transport through the real embedding into `Complex`.
theorem complex_from_real_cauchy_metric(q: Nat -> Real) {
    is_cauchy_metric(q) implies is_cauchy_metric(compose(Complex.from_real, q))
} by {
    if is_cauchy_metric(q) {
        complex_from_real_is_isometry
        isometry_preserves_cauchy_metric(Complex.from_real, q)
        is_cauchy_metric(compose(Complex.from_real, q))
    }
}

/// Classical real convergence transports to metric convergence after embedding into `Complex`.
theorem complex_from_real_converges_to_tendsto_metric(q: Nat -> Real, a: Real) {
    converges_to(q, a) implies tendsto_metric(compose(Complex.from_real, q), Complex.from_real(a))
} by {
    if converges_to(q, a) {
        real_converges_to_imp_tendsto_metric(q, a)
        complex_from_real_tendsto_metric(q, a)
        tendsto_metric(compose(Complex.from_real, q), Complex.from_real(a))
    }
}
