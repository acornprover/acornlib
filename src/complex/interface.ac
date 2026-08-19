/// Public interface for complex numbers.

from real import Real, two, from_nat, converges, converges_to
from algebra.add import Add
from algebra.mul import Mul
from algebra.neg import Neg
from algebra.zero import Zero
from algebra.one import One
from algebra.inverse import Inverse
from algebra.add_semigroup import AddSemigroup
from algebra.add_comm_semigroup import AddCommSemigroup
from algebra.add_monoid import AddMonoid
from algebra.add_comm_monoid import AddCommMonoid
from algebra.semigroup import Semigroup
from algebra.monoid.monoid import Monoid
from algebra.add_group import AddGroup
from algebra.add_comm_group import AddCommGroup
from algebra.comm_semigroup import CommSemigroup
from algebra.comm_monoid import CommMonoid
from algebra.ring.ring import Ring
from algebra.field.field import Field
from algebra.field.field_hom import FieldHom
from comm_ring import CommRing
from semiring import Semiring
from rat import Rat
from nat import Nat
from list import partial
from polynomial import Polynomial, polynomial_constant, polynomial_monomial, polynomial_eval, polynomial_support_bounded_by
/// Complex numbers consist of a real part and an imaginary part.
/// They extend the real numbers and satisfy the equation i² = -1.
structure Complex {
    /// The real part of the complex number.
    re: Real
    /// The imaginary part of the complex number.
    im: Real
}



attributes Complex {
    /// Converts a real number to a complex number (with zero imaginary part).
    let from_real: Real -> Complex = function(r: Real) {
        Complex.new(r, Real.0)
    }

    /// The imaginary unit, satisfying `i² = -1`.
    let i: Complex = Complex.new(Real.0, Real.1)

    /// True if this complex number has no imaginary component.
    define is_real(self) -> Bool {
        self.im = Real.0
    }

    /// True if this complex number is purely imaginary (no real component).
    define is_imaginary(self) -> Bool {
        self.re = Real.0 and self.im != Real.0
    }

    /// Yields the complex conjugate (negates the imaginary part).
    define conj(self) -> Complex {
        Complex.new(self.re, -self.im)
    }

    /// Computes the squared magnitude |z|² = re² + im².
    define abs_squared(self) -> Real {
        self.re * self.re + self.im * self.im
    }

}
/// Adds two complex numbers component-wise.
instance Complex: Add {
    define add(self, other: Complex) -> Complex {
        Complex.new(self.re + other.re, self.im + other.im)
    }
}

/// Multiplies two complex numbers using the formula (a+bi)(c+di) = (ac-bd)+(ad+bc)i.
instance Complex: Mul {
    define mul(self, other: Complex) -> Complex {
        Complex.new(
            self.re * other.re - self.im * other.im,
            self.re * other.im + self.im * other.re
        )
    }
}

instance Complex: Zero {
    let 0: Complex = Complex.new(Real.0, Real.0)
}

instance Complex: One {
    let 1: Complex = Complex.new(Real.1, Real.0)
}

instance Complex: Neg {
    define neg(self) -> Complex {
        Complex.new(-self.re, -self.im)
    }
}

instance Complex: Inverse {
    define inverse(self) -> Complex {
        self.conj * Complex.from_real(self.abs_squared.inverse)
    }
}


attributes Complex {
    /// Divides this complex number by another.
    /// Division by zero returns zero (making this a total function).
    define div(self, other: Complex) -> Complex {
        self * other.inverse
    }
}

instance Complex: AddSemigroup
instance Complex: AddCommSemigroup
instance Complex: AddMonoid
instance Complex: AddCommMonoid
instance Complex: Semigroup
instance Complex: Monoid
instance Complex: Semiring
instance Complex: AddGroup
instance Complex: AddCommGroup
instance Complex: Ring
instance Complex: CommSemigroup
instance Complex: CommMonoid
instance Complex: CommRing
instance Complex: Field

/// The modulus |z| of a complex number, the nonnegative square root of `abs_squared`.
let complex_modulus(z: Complex) -> r: Real satisfy {
    z.abs_squared.sqrt = Option.some(r) and r * r = z.abs_squared and r >= Real.0
}

attributes Complex {
    /// The modulus |z| of the complex number.
    let modulus: Complex -> Real = complex_modulus
}

/// The nth term in the complex exponential series.
define complex_exp_term(z: Complex, n: Nat) -> Complex {
    z.pow(n) / Complex.from_real(Real.from_rat(Rat.from_nat(n.factorial)))
}

/// The sequence of real parts of a complex sequence.
define re_seq(z: Nat -> Complex, n: Nat) -> Real {
    z(n).re
}

/// The sequence of imaginary parts of a complex sequence.
define im_seq(z: Nat -> Complex, n: Nat) -> Real {
    z(n).im
}

/// A complex sequence converges to a limit iff both component sequences do.
define complex_converges_to(z: Nat -> Complex, w: Complex) -> Bool {
    converges_to(re_seq(z), w.re) and converges_to(im_seq(z), w.im)
}

/// A complex sequence converges iff some limit exists.
define complex_converges(z: Nat -> Complex) -> Bool {
    exists(w: Complex) {
        complex_converges_to(z, w)
    }
}

/// The limit of a complex sequence, with a default of zero for divergent sequences.
let complex_limit(z: Nat -> Complex) -> lim: Complex satisfy {
    (complex_converges(z) implies complex_converges_to(z, lim)) and
    (not complex_converges(z) implies lim = Complex.new(Real.0, Real.0))
}

/// The real-component function.
let complex_re_fn: Complex -> Real = function(c: Complex) {
    c.re
}

/// The imaginary-component function.
let complex_im_fn: Complex -> Real = function(c: Complex) {
    c.im
}

/// The complex exponential function z.exp = Σ z^n / n!,
/// defined as the limit of the partial sums of the complex exponential series.
define complex_exp(z: Complex) -> Complex {
    complex_limit(partial(complex_exp_term(z)))
}

/// The complex logarithm, using the principal branch.
define complex_log(z: Complex) -> Complex {
    Complex.from_real((z.modulus).log.get_or_else(Real.0))
}

/// The complex cosine function: z.cos = (exp(i·z) + exp(-i·z)) / 2.
define complex_cos(z: Complex) -> Complex {
    (complex_exp(Complex.i * z) + complex_exp(-(Complex.i * z))) / Complex.from_real(two)
}

/// The complex sine function: z.sin = (exp(i·z) - exp(-i·z)) / (2·i).
define complex_sin(z: Complex) -> Complex {
    (complex_exp(Complex.i * z) - complex_exp(-(Complex.i * z))) / (Complex.from_real(two) * Complex.i)
}

attributes Complex {
    /// The complex exponential function.
    let exp = complex_exp

    /// The complex logarithm.
    let log = complex_log

    /// The complex cosine function.
    let cos = complex_cos

    /// The complex sine function.
    let sin = complex_sin
}

/// The imaginary unit times a real number.
define complex_i_mul_real(x: Real) -> Complex {
    Complex.i * Complex.from_real(x)
}

/// The quadratic polynomial a·X² + b·X + c.
define quadratic_polynomial(a: Complex, b: Complex, c: Complex) -> Polynomial[Complex] {
    polynomial_constant(c) + polynomial_monomial(Nat.1, b) + polynomial_monomial(Nat.2, a)
}

/// A theorem about complex numbers.
theorem abs_squared_add(a: Complex, b: Complex) {
    (a + b).abs_squared = a.abs_squared + b.abs_squared + ((a * b.conj).re + (a * b.conj).re)
}

/// A theorem about complex numbers.
theorem abs_squared_conj(a: Complex) {
    a * a.conj = Complex.new(a.abs_squared, Real.0)
}

/// A theorem about complex numbers.
theorem abs_squared_eq(a: Complex) {
    a.abs_squared = a.re * a.re + a.im * a.im
}

/// A theorem about complex numbers.
theorem abs_squared_mul(a: Complex, b: Complex) {
    (a * b).abs_squared = a.abs_squared * b.abs_squared
}

/// A theorem about complex numbers.
theorem abs_squared_neg(a: Complex) {
    (-a).abs_squared = a.abs_squared
}

/// A theorem about complex numbers.
theorem add_assoc(a: Complex, b: Complex, c: Complex) {
    (a + b) + c = a + (b + c)
}

/// A theorem about complex numbers.
theorem add_comm(a: Complex, b: Complex) {
    a + b = b + a
}

/// A theorem about complex numbers.
theorem add_neg_cancel(a: Complex) {
    a + -a = Complex.0
}

/// A theorem about complex numbers.
theorem add_zero_left(a: Complex) {
    Complex.0 + a = a
}

/// A theorem about complex numbers.
theorem add_zero_right(a: Complex) {
    a + Complex.0 = a
}

/// The complex conjugation function.
let complex_conj_fn: Complex -> Complex = function(c: Complex) {
    c.conj
}

/// Complex conjugation as a field homomorphism of the complex numbers.
let complex_conj_field_hom: FieldHom[Complex, Complex] satisfy {
    FieldHom.new(complex_conj_fn) = Option.some(complex_conj_field_hom)
}

/// A theorem about complex numbers.
theorem complex_conj_field_hom_from_real(a: Real) {
    complex_conj_field_hom.hom(Complex.from_real(a)) = Complex.from_real(a)
}

/// A theorem about complex numbers.
theorem complex_euler_formula(x: Real) {
    complex_exp(Complex.i * Complex.from_real(x)) =
        complex_cos(Complex.from_real(x)) + Complex.i * complex_sin(Complex.from_real(x))
}

/// A theorem about complex numbers.
theorem complex_exp_pow(z: Complex, n: Nat) {
    complex_exp(z).pow(n) = complex_exp(Complex.from_real(from_nat[Real](n)) * z)
}

/// A theorem about complex numbers.
theorem complex_i_mul_real_mul(r: Real, x: Real) {
    Complex.from_real(r) * complex_i_mul_real(x) = complex_i_mul_real(r * x)
}

/// A theorem about complex numbers.
theorem complex_im_fn_add(a: Complex, b: Complex) {
    complex_im_fn(a + b) = complex_im_fn(a) + complex_im_fn(b)
}

/// A theorem about complex numbers.
theorem complex_re_fn_add(a: Complex, b: Complex) {
    complex_re_fn(a + b) = complex_re_fn(a) + complex_re_fn(b)
}

/// A theorem about complex numbers.
theorem conj_abs_squared(z: Complex) {
    z.conj.abs_squared = z.abs_squared
}

/// A theorem about complex numbers.
theorem conj_conj(a: Complex) {
    a.conj.conj = a
}

/// A theorem about complex numbers.
theorem conj_mul(a: Complex, b: Complex) {
    (a * b).conj = a.conj * b.conj
}

/// A theorem about complex numbers.
theorem conj_neg(a: Complex) {
    (-a).conj = -(a.conj)
}

/// A theorem about complex numbers.
theorem distrib(a: Complex, b: Complex, c: Complex) {
    a * (b + c) = a * b + a * c
}

/// A theorem about complex numbers.
theorem eq_by_components(a: Complex, b: Complex) {
    a.re = b.re and a.im = b.im implies a = b
}

/// A theorem about complex numbers.
theorem fundamental_theorem_of_algebra_of_extreme(p: Polynomial[Complex]) {
    exists(x: Complex, y: Complex) { polynomial_eval(p, x) != polynomial_eval(p, y) } and
    exists(z0: Complex) {
        forall(z: Complex) { polynomial_eval(p, z0).modulus <= polynomial_eval(p, z).modulus }
    }
    implies exists(z: Complex) { polynomial_eval(p, z) = Complex.0 }
}

/// A theorem about complex numbers.
theorem i_squared_eq_neg_one {
    Complex.i * Complex.i = -Complex.1
}

/// A theorem about complex numbers.
theorem im_add(a: Complex, b: Complex) {
    (a + b).im = a.im + b.im
}

/// A theorem about complex numbers.
theorem im_conj(a: Complex) {
    a.conj.im = -a.im
}

/// A theorem about complex numbers.
theorem im_from_real(a: Real) {
    Complex.from_real(a).im = Real.0
}

/// A theorem about complex numbers.
theorem im_i {
    Complex.i.im = Real.1
}

/// A theorem about complex numbers.
theorem im_mul(a: Complex, b: Complex) {
    (a * b).im = a.re * b.im + a.im * b.re
}

/// A theorem about complex numbers.
theorem im_one {
    Complex.1.im = Real.0
}

/// A theorem about complex numbers.
theorem im_sub(a: Complex, b: Complex) {
    (a - b).im = a.im - b.im
}

/// A theorem about complex numbers.
theorem im_zero {
    Complex.0.im = Real.0
}

/// A theorem about complex numbers.
theorem left_distrib(a: Complex, b: Complex, c: Complex) {
    (a + b) * c = a * c + b * c
}

/// A theorem about complex numbers.
theorem modulus_conj(z: Complex) {
    z.conj.modulus = z.modulus
}

/// A theorem about complex numbers.
theorem modulus_mul(a: Complex, b: Complex) {
    (a * b).modulus = a.modulus * b.modulus
}

/// A theorem about complex numbers.
theorem modulus_squared(z: Complex) {
    z.modulus * z.modulus = z.abs_squared
}

/// A theorem about complex numbers.
theorem mul_assoc(a: Complex, b: Complex, c: Complex) {
    (a * b) * c = a * (b * c)
}

/// A theorem about complex numbers.
theorem mul_comm(a: Complex, b: Complex) {
    a * b = b * a
}

/// A theorem about complex numbers.
theorem mul_conj_im(a: Complex) {
    (a * a.conj).im = Real.0
}

/// A theorem about complex numbers.
theorem mul_conj_re(a: Complex) {
    (a * a.conj).re = a.abs_squared
}

/// A theorem about complex numbers.
theorem mul_from_real(c: Complex, r: Real) {
    c * Complex.from_real(r) = Complex.new(c.re * r, c.im * r)
}

/// A theorem about complex numbers.
theorem mul_left_cancel(a: Complex, b: Complex, c: Complex) {
    a * b = a * c and a != Complex.0 implies b = c
}

/// A theorem about complex numbers.
theorem mul_one_left(a: Complex) {
    Complex.1 * a = a
}

/// A theorem about complex numbers.
theorem mul_one_right(a: Complex) {
    a * Complex.1 = a
}

/// A theorem about complex numbers.
theorem mul_zero_left(a: Complex) {
    Complex.0 * a = Complex.0
}

/// A theorem about complex numbers.
theorem mul_zero_right(a: Complex) {
    a * Complex.0 = Complex.0
}

/// A theorem about complex numbers.
theorem neg_im(a: Complex) {
    -(a.im) = (-a).im
}

/// A theorem about complex numbers.
theorem neg_re(a: Complex) {
    -(a.re) = (-a).re
}

/// A theorem about complex numbers.
theorem polynomial_eval_monomial_one(r: Complex, x: Complex) {
    polynomial_eval(polynomial_monomial(Nat.1, r), x) = r * x
}

/// A theorem about complex numbers.
theorem polynomial_eval_monomial_two(r: Complex, x: Complex) {
    polynomial_eval(polynomial_monomial(Nat.2, r), x) = r * x * x
}

/// A theorem about complex numbers.
theorem polynomial_monomial_support_bounded_by_suc[R: Semiring](n: Nat, r: R) {
    polynomial_support_bounded_by(Polynomial[R].monomial(n, r), n.suc)
}

/// A theorem about complex numbers.
theorem re_add(a: Complex, b: Complex) {
    (a + b).re = a.re + b.re
}

/// A theorem about complex numbers.
theorem re_conj(a: Complex) {
    a.conj.re = a.re
}

/// A theorem about complex numbers.
theorem re_from_real(a: Real) {
    Complex.from_real(a).re = a
}

/// A theorem about complex numbers.
theorem re_i {
    Complex.i.re = Real.0
}

/// A theorem about complex numbers.
theorem re_mul(a: Complex, b: Complex) {
    (a * b).re = a.re * b.re - a.im * b.im
}

/// A theorem about complex numbers.
theorem re_neg(a: Complex) {
    (-a).re = -a.re
}

/// A theorem about complex numbers.
theorem re_one {
    Complex.1.re = Real.1
}

/// A theorem about complex numbers.
theorem re_sub(a: Complex, b: Complex) {
    (a - b).re = a.re - b.re
}

/// A theorem about complex numbers.
theorem re_zero {
    Complex.0.re = Real.0
}

/// A theorem about complex numbers.
theorem real_add_lifts(a: Real, b: Real) {
    Complex.from_real(a + b) = Complex.from_real(a) + Complex.from_real(b)
}

/// A theorem about complex numbers.
theorem real_mul_lifts(a: Real, b: Real) {
    Complex.from_real(a * b) = Complex.from_real(a) * Complex.from_real(b)
}
