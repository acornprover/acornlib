// ---------------------------------------------------------------------------
// The complex geometric series.
//
// This module proves the fundamental convergence criterion for the complex
// geometric series:
//
//   The series sum z^n converges for every complex z with |z| < 1.
//
// The proof bounds the real and imaginary parts of the terms by |z|^n, so the
// real and imaginary partial sums converge by comparison with the real
// geometric series, and the complex partial sums converge componentwise.
// ---------------------------------------------------------------------------

from real import Real, converges, converges_to, limit, abs_fn, absolutely_converges,
    absolutely_converges_imp_converges, geom_converges, comparison_test, is_lower_bound,
    seq_lte, abs_gte_zero, pos_imp_eq_abs, converges_imp_converges_to
from nat import Nat
from list import partial
from complex.complex import Complex
from complex.complex_abs import re_abs_le_modulus, im_abs_le_modulus, modulus_nonneg, modulus_of_zero
from complex.complex_exp import complex_modulus_pow, partial_re_seq, partial_im_seq, real_abs_zero
from complex.complex_seq import re_seq, im_seq, complex_converges, complex_converges_to,
    componentwise_imp_complex_converges_to

/// The absolute value of the real part of a geometric term is bounded by the
/// geometric term: |Re(z^n)| <= |z|^n.
theorem complex_geom_re_abs_le(z: Complex, n: Nat) {
    z.pow(n).re.abs <= z.modulus.pow(n)
} by {
    re_abs_le_modulus(z.pow(n))
    z.pow(n).re.abs <= z.pow(n).modulus
    complex_modulus_pow(z, n)
    z.pow(n).modulus = z.modulus.pow(n)
    z.pow(n).re.abs <= z.modulus.pow(n)
}

/// The absolute value of the imaginary part of a geometric term is bounded by
/// the geometric term: |Im(z^n)| <= |z|^n.
theorem complex_geom_im_abs_le(z: Complex, n: Nat) {
    z.pow(n).im.abs <= z.modulus.pow(n)
} by {
    im_abs_le_modulus(z.pow(n))
    z.pow(n).im.abs <= z.pow(n).modulus
    complex_modulus_pow(z, n)
    z.pow(n).modulus = z.modulus.pow(n)
    z.pow(n).im.abs <= z.modulus.pow(n)
}

/// The complex geometric series converges for |z| < 1.
theorem complex_geom_series_converges(z: Complex) {
    z.modulus < Real.1 implies complex_converges(partial(z.pow))
} by {
    if z.modulus < Real.1 {
        modulus_nonneg(z)
        z.modulus >= Real.0
        if z.modulus = Real.0 {
            real_abs_zero
            Real.0.abs = Real.0
            z.modulus.abs = Real.0
            z.modulus.abs = z.modulus
        } else {
            z.modulus != Real.0
            z.modulus > Real.0
            z.modulus.is_positive
            pos_imp_eq_abs(z.modulus)
            z.modulus = z.modulus.abs
            z.modulus.abs = z.modulus
        }
        z.modulus.abs < Real.1
        geom_converges(z.modulus)
        z.modulus.abs < Real.1 implies converges(partial(z.modulus.pow))
        converges(partial(z.modulus.pow))
        forall(n: Nat) {
            abs_gte_zero(re_seq(z.pow, n))
            re_seq(z.pow, n).abs >= Real.0
            Real.0 <= re_seq(z.pow, n).abs
            abs_fn(re_seq(z.pow), n) = re_seq(z.pow, n).abs
            Real.0 <= abs_fn(re_seq(z.pow), n)
        }
        is_lower_bound(abs_fn(re_seq(z.pow)), Real.0)
        forall(n: Nat) {
            abs_fn(re_seq(z.pow), n) = re_seq(z.pow, n).abs
            re_seq(z.pow, n) = z.pow(n).re
            abs_fn(re_seq(z.pow), n) = z.pow(n).re.abs
            complex_geom_re_abs_le(z, n)
            z.pow(n).re.abs <= z.modulus.pow(n)
            abs_fn(re_seq(z.pow), n) <= z.modulus.pow(n)
        }
        seq_lte(abs_fn(re_seq(z.pow)), z.modulus.pow)
        comparison_test(abs_fn(re_seq(z.pow)), z.modulus.pow)
        is_lower_bound(abs_fn(re_seq(z.pow)), Real.0) and seq_lte(abs_fn(re_seq(z.pow)), z.modulus.pow) and converges(partial(z.modulus.pow)) implies converges(partial(abs_fn(re_seq(z.pow))))
        converges(partial(abs_fn(re_seq(z.pow))))
        absolutely_converges(re_seq(z.pow))
        absolutely_converges_imp_converges(re_seq(z.pow))
        converges(partial(re_seq(z.pow)))
        forall(n: Nat) {
            abs_gte_zero(im_seq(z.pow, n))
            im_seq(z.pow, n).abs >= Real.0
            Real.0 <= im_seq(z.pow, n).abs
            abs_fn(im_seq(z.pow), n) = im_seq(z.pow, n).abs
            Real.0 <= abs_fn(im_seq(z.pow), n)
        }
        is_lower_bound(abs_fn(im_seq(z.pow)), Real.0)
        forall(n: Nat) {
            abs_fn(im_seq(z.pow), n) = im_seq(z.pow, n).abs
            im_seq(z.pow, n) = z.pow(n).im
            abs_fn(im_seq(z.pow), n) = z.pow(n).im.abs
            complex_geom_im_abs_le(z, n)
            z.pow(n).im.abs <= z.modulus.pow(n)
            abs_fn(im_seq(z.pow), n) <= z.modulus.pow(n)
        }
        seq_lte(abs_fn(im_seq(z.pow)), z.modulus.pow)
        comparison_test(abs_fn(im_seq(z.pow)), z.modulus.pow)
        is_lower_bound(abs_fn(im_seq(z.pow)), Real.0) and seq_lte(abs_fn(im_seq(z.pow)), z.modulus.pow) and converges(partial(z.modulus.pow)) implies converges(partial(abs_fn(im_seq(z.pow))))
        converges(partial(abs_fn(im_seq(z.pow))))
        absolutely_converges(im_seq(z.pow))
        absolutely_converges_imp_converges(im_seq(z.pow))
        converges(partial(im_seq(z.pow)))
        converges_imp_converges_to(partial(re_seq(z.pow)))
        converges_to(partial(re_seq(z.pow)), limit(partial(re_seq(z.pow))))
        forall(n: Nat) {
            partial_re_seq(z.pow, n)
            re_seq(partial(z.pow), n) = partial(re_seq(z.pow), n)
        }
        re_seq(partial(z.pow)) = partial(re_seq(z.pow))
        converges_to(re_seq(partial(z.pow)), limit(partial(re_seq(z.pow))))
        converges_imp_converges_to(partial(im_seq(z.pow)))
        converges_to(partial(im_seq(z.pow)), limit(partial(im_seq(z.pow))))
        forall(n: Nat) {
            partial_im_seq(z.pow, n)
            im_seq(partial(z.pow), n) = partial(im_seq(z.pow), n)
        }
        im_seq(partial(z.pow)) = partial(im_seq(z.pow))
        converges_to(im_seq(partial(z.pow)), limit(partial(im_seq(z.pow))))
        componentwise_imp_complex_converges_to(partial(z.pow),
            Complex.new(limit(partial(re_seq(z.pow))), limit(partial(im_seq(z.pow)))))
        converges_to(re_seq(partial(z.pow)), Complex.new(limit(partial(re_seq(z.pow))), limit(partial(im_seq(z.pow)))).re) and converges_to(im_seq(partial(z.pow)), Complex.new(limit(partial(re_seq(z.pow))), limit(partial(im_seq(z.pow)))).im) implies complex_converges_to(partial(z.pow), Complex.new(limit(partial(re_seq(z.pow))), limit(partial(im_seq(z.pow)))))
        converges_to(re_seq(partial(z.pow)), Complex.new(limit(partial(re_seq(z.pow))), limit(partial(im_seq(z.pow)))).re)
        converges_to(im_seq(partial(z.pow)), Complex.new(limit(partial(re_seq(z.pow))), limit(partial(im_seq(z.pow)))).im)
        complex_converges_to(partial(z.pow), Complex.new(limit(partial(re_seq(z.pow))), limit(partial(im_seq(z.pow)))))
        complex_converges(partial(z.pow))
    }
}

/// The geometric series at zero converges.
theorem complex_geom_series_zero_converges {
    complex_converges(partial(Complex.0.pow))
} by {
    modulus_of_zero
    Complex.0.modulus = Real.0
    Real.0 < Real.1
    complex_geom_series_converges(Complex.0)
    Complex.0.modulus < Real.1 implies complex_converges(partial(Complex.0.pow))
    complex_converges(partial(Complex.0.pow))
}

