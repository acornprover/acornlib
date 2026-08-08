from real import Real
from algebra.module.module_hom import is_linear_map, preserves_add, preserves_smul, ModuleHom,
    is_linear_equiv
from data.basic.functions import is_two_sided_inverse_fn, is_left_inverse_fn, is_right_inverse_fn,
    is_bijection_fn, two_sided_inverse_fn_imp_bijection_fn
from complex.complex import Complex, complex_real_smul, conj_conj
from complex.complex_conj_hom import complex_conj_fn, complex_conj_fn_add
from complex.complex_conj_linear import complex_conj_fn_real_smul
from complex.complex_module import complex_real_module, complex_real_module_smul
from algebra.module.module_iso import is_linear_equiv_pair, modules_linearly_equivalent,
    modules_linearly_equivalent_of_pair, LinearEquiv

/// Complex conjugation preserves addition.
theorem complex_conj_fn_preserves_add {
    preserves_add(complex_conj_fn)
} by {
    forall(a: Complex, b: Complex) {
        complex_conj_fn_add(a, b)
    }
}

/// Helper: per-input form of the real-scalar preservation equality for `complex_conj_fn`.
theorem complex_conj_fn_smul_eq(r: Real, x: Complex) {
    complex_conj_fn(complex_real_module.smul(r, x)) = complex_real_module.smul(r, complex_conj_fn(x))
} by {
    complex_real_module_smul(r, x)
    complex_conj_fn_real_smul(r, x)
    complex_real_module_smul(r, complex_conj_fn(x))
}

/// Complex conjugation preserves the real scalar action.
theorem complex_conj_fn_preserves_smul {
    preserves_smul(complex_real_module, complex_real_module, complex_conj_fn)
} by {
    forall(r: Real, x: Complex) {
        complex_conj_fn_smul_eq(r, x)
    }
}

/// Complex conjugation is a real-linear map from the complex numbers to themselves.
theorem complex_conj_fn_is_linear_map {
    is_linear_map(complex_real_module, complex_real_module, complex_conj_fn)
} by {
    complex_conj_fn_preserves_add
    complex_conj_fn_preserves_smul
}

/// `ModuleHom.new` produces a `Some` value for complex conjugation.
theorem complex_conj_fn_module_hom_some {
    exists(h: ModuleHom[Real, Complex, Complex]) {
        ModuleHom.new(complex_real_module, complex_real_module, complex_conj_fn) = Option.some(h)
    }
} by {
    complex_conj_fn_is_linear_map
}

/// Complex conjugation packaged as a real-linear `ModuleHom`.
let complex_conj_module_hom: ModuleHom[Real, Complex, Complex] satisfy {
    ModuleHom.new(complex_real_module, complex_real_module, complex_conj_fn) = Option.some(complex_conj_module_hom)
}

/// The source module of `complex_conj_module_hom` is `complex_real_module`.
theorem complex_conj_module_hom_src {
    complex_conj_module_hom.src = complex_real_module
} by {
    ModuleHom.new(complex_real_module, complex_real_module, complex_conj_fn) = Option.some(complex_conj_module_hom)
}

/// The destination module of `complex_conj_module_hom` is `complex_real_module`.
theorem complex_conj_module_hom_dst {
    complex_conj_module_hom.dst = complex_real_module
} by {
    ModuleHom.new(complex_real_module, complex_real_module, complex_conj_fn) = Option.some(complex_conj_module_hom)
}


/// Complex conjugation is its own two-sided inverse.
theorem complex_conj_fn_two_sided_inverse {
    is_two_sided_inverse_fn(complex_conj_fn, complex_conj_fn)
} by {
    forall(x: Complex) {
        complex_conj_fn(complex_conj_fn(x)) = x.conj.conj
        conj_conj(x)
    }
    is_left_inverse_fn(complex_conj_fn, complex_conj_fn)
    is_right_inverse_fn(complex_conj_fn, complex_conj_fn)
}

/// Complex conjugation is a bijection of the complex numbers.
theorem complex_conj_fn_is_bijection {
    is_bijection_fn(complex_conj_fn)
} by {
    complex_conj_fn_two_sided_inverse
    two_sided_inverse_fn_imp_bijection_fn(complex_conj_fn, complex_conj_fn)
}

/// Complex conjugation is a real-linear equivalence of the complex numbers.
theorem complex_conj_fn_is_linear_equiv {
    is_linear_equiv(complex_real_module, complex_real_module, complex_conj_fn)
} by {
    complex_conj_fn_is_linear_map
    complex_conj_fn_is_bijection
}

/// Complex conjugation pairs with itself as a real-linear equivalence of the complex
/// numbers.
theorem complex_conj_fn_is_linear_equiv_pair {
    is_linear_equiv_pair(complex_real_module, complex_real_module, complex_conj_fn, complex_conj_fn)
} by {
    complex_conj_fn_is_linear_map
    complex_conj_fn_two_sided_inverse
}

/// `LinearEquiv.new` produces a `Some` value for complex conjugation paired with itself.
theorem complex_conj_linear_equiv_some {
    exists(h: LinearEquiv[Real, Complex, Complex]) {
        LinearEquiv.new(complex_real_module, complex_real_module, complex_conj_fn, complex_conj_fn) =
            Option.some(h)
    }
} by {
    complex_conj_fn_is_linear_map
    complex_conj_fn_two_sided_inverse
    is_linear_map(complex_real_module, complex_real_module, complex_conj_fn) and
        is_two_sided_inverse_fn(complex_conj_fn, complex_conj_fn)
}

/// Complex conjugation packaged as a bundled real-linear self-equivalence of the
/// complex numbers, with itself as its two-sided inverse.
let complex_conj_linear_equiv: LinearEquiv[Real, Complex, Complex] satisfy {
    LinearEquiv.new(complex_real_module, complex_real_module, complex_conj_fn, complex_conj_fn) =
        Option.some(complex_conj_linear_equiv)
}

/// The source of the conjugation linear equivalence is the real module of complex numbers.
theorem complex_conj_linear_equiv_src {
    complex_conj_linear_equiv.src = complex_real_module
} by {
    LinearEquiv.new(complex_real_module, complex_real_module, complex_conj_fn, complex_conj_fn) =
        Option.some(complex_conj_linear_equiv)
}

/// The destination of the conjugation linear equivalence is the real module of complex numbers.
theorem complex_conj_linear_equiv_dst {
    complex_conj_linear_equiv.dst = complex_real_module
} by {
    LinearEquiv.new(complex_real_module, complex_real_module, complex_conj_fn, complex_conj_fn) =
        Option.some(complex_conj_linear_equiv)
}

/// The complex numbers, as a real module, are linearly equivalent to themselves via
/// complex conjugation.
theorem complex_real_module_linearly_equivalent_self {
    modules_linearly_equivalent(complex_real_module, complex_real_module)
} by {
    complex_conj_fn_is_linear_equiv_pair
    modules_linearly_equivalent_of_pair(
        complex_real_module, complex_real_module, complex_conj_fn, complex_conj_fn)
}
