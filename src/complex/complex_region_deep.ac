// ---------------------------------------------------------------------------
// Deepening of the theory of complex regions.
//
// This module extends src/complex/complex_region.ac with the closure laws of
// the unit disk:
//
//   a. The closed unit disk is closed under multiplication and under natural
//      powers.
//   b. The special points 0, 1, -1 and i lie in the unit disk, and the unit
//      circle lies inside it.
//   c. Every root of unity lies in the unit disk, and zero lies in the open
//      unit disk.
// ---------------------------------------------------------------------------

from real import Real, two, pi, mul_le_mul_of_nonneg_right, mul_le_mul_of_nonneg_left
from nat import Nat, pow_zero, alt_induction
from order import lte_trans, lt_imp_lte
from complex.complex import Complex, from_real_one
from complex.complex_abs import modulus_mul, modulus_of_one, modulus_of_i, modulus_neg,
    modulus_of_zero, modulus_nonneg
from complex.complex_abs_deep import modulus_sub_self_zero
from complex.complex_exp import real_pow_suc, real_mul_one_left, real_mul_one_right,
    complex_modulus_pow, real_one_positive
from complex.complex_exp_deep import complex_exp_unit_modulus
from complex.complex_region import unit_disk, in_closed_disk, closed_disk, open_disk,
    unit_disk_contains_eq, open_disk_contains_eq
from complex.complex_unit_circle import is_unit, unit_omega
from complex.roots_of_unity import omega

/// A nonnegative number below one has all powers below one: x^n <= 1.
theorem real_pow_le_one(x: Real, n: Nat) {
    x >= Real.0 and x <= Real.1 implies x.pow(n) <= Real.1
} by {
    define p(k: Nat) -> Bool {
        x.pow(k) <= Real.1
    }
    if x >= Real.0 and x <= Real.1 {
        pow_zero(x)
        x.pow(Nat.0) = Real.1
        Real.1 <= Real.1
        x.pow(Nat.0) <= Real.1
        p(Nat.0)
        forall(k: Nat) {
            if p(k) {
                x.pow(k) <= Real.1
                real_pow_suc(x, k)
                x.pow(k.suc) = x * x.pow(k)
                mul_le_mul_of_nonneg_left[Real](x.pow(k), Real.1, x)
                x.pow(k) <= Real.1 and Real.0 <= x implies x * x.pow(k) <= x * Real.1
                x * x.pow(k) <= x * Real.1
                real_mul_one_right(x)
                x * Real.1 = x
                x * x.pow(k) <= x
                x.pow(k.suc) <= x
                x <= Real.1
                lte_trans(x.pow(k.suc), x, Real.1)
                x.pow(k.suc) <= Real.1
                p(k.suc)
            }
        }
        p(Nat.0) and forall(k: Nat) { p(k) implies p(k.suc) }
        alt_induction(p)
        p(n)
    }
}

/// The closed unit disk is closed under multiplication.
theorem unit_disk_mul_closed(z: Complex, w: Complex) {
    unit_disk.contains(z) and unit_disk.contains(w) implies unit_disk.contains(z * w)
} by {
    if unit_disk.contains(z) and unit_disk.contains(w) {
        unit_disk_contains_eq(z)
        unit_disk.contains(z) = (z.modulus <= Real.1)
        z.modulus <= Real.1
        unit_disk_contains_eq(w)
        unit_disk.contains(w) = (w.modulus <= Real.1)
        w.modulus <= Real.1
        modulus_mul(z, w)
        (z * w).modulus = z.modulus * w.modulus
        modulus_nonneg(w)
        w.modulus >= Real.0
        mul_le_mul_of_nonneg_right[Real](z.modulus, Real.1, w.modulus)
        z.modulus <= Real.1 and Real.0 <= w.modulus implies z.modulus * w.modulus <= Real.1 * w.modulus
        z.modulus * w.modulus <= Real.1 * w.modulus
        real_mul_one_left(w.modulus)
        Real.1 * w.modulus = w.modulus
        z.modulus * w.modulus <= w.modulus
        lte_trans(z.modulus * w.modulus, w.modulus, Real.1)
        z.modulus * w.modulus <= Real.1
        (z * w).modulus <= Real.1
        unit_disk_contains_eq(z * w)
        unit_disk.contains(z * w)
    }
}

/// The closed unit disk is closed under natural powers.
theorem unit_disk_pow_closed(z: Complex, n: Nat) {
    unit_disk.contains(z) implies unit_disk.contains(z.pow(n))
} by {
    if unit_disk.contains(z) {
        unit_disk_contains_eq(z)
        unit_disk.contains(z) = (z.modulus <= Real.1)
        z.modulus <= Real.1
        modulus_nonneg(z)
        z.modulus >= Real.0
        real_pow_le_one(z.modulus, n)
        z.modulus >= Real.0 and z.modulus <= Real.1 implies z.modulus.pow(n) <= Real.1
        z.modulus.pow(n) <= Real.1
        complex_modulus_pow(z, n)
        z.pow(n).modulus = z.modulus.pow(n)
        z.pow(n).modulus <= Real.1
        unit_disk_contains_eq(z.pow(n))
        unit_disk.contains(z.pow(n))
    }
}

/// The unit circle lies inside the closed unit disk.
theorem unit_circle_in_unit_disk(z: Complex) {
    is_unit(z) implies unit_disk.contains(z)
} by {
    if is_unit(z) {
        z.modulus = Real.1
        z.modulus <= Real.1
        unit_disk_contains_eq(z)
        unit_disk.contains(z)
    }
}

/// Zero lies in the closed unit disk.
theorem unit_disk_contains_zero {
    unit_disk.contains(Complex.0)
} by {
    modulus_of_zero
    Complex.0.modulus = Real.0
    real_one_positive
    Real.1 > Real.0
    Real.0 < Real.1
    lt_imp_lte(Real.0, Real.1)
    Real.0 <= Real.1
    unit_disk_contains_eq(Complex.0)
    unit_disk.contains(Complex.0)
}

/// One lies in the closed unit disk.
theorem unit_disk_contains_one {
    unit_disk.contains(Complex.1)
} by {
    modulus_of_one
    Complex.1.modulus = Real.1
    Real.1 <= Real.1
    unit_disk_contains_eq(Complex.1)
    unit_disk.contains(Complex.1)
}

/// Negative one lies in the closed unit disk.
theorem unit_disk_contains_neg_one {
    unit_disk.contains(-Complex.1)
} by {
    modulus_neg(Complex.1)
    (-Complex.1).modulus = Complex.1.modulus
    modulus_of_one
    Complex.1.modulus = Real.1
    (-Complex.1).modulus = Real.1
    Real.1 <= Real.1
    unit_disk_contains_eq(-Complex.1)
    unit_disk.contains(-Complex.1)
}

/// The imaginary unit lies in the closed unit disk.
theorem unit_disk_contains_i {
    unit_disk.contains(Complex.i)
} by {
    modulus_of_i
    Complex.i.modulus = Real.1
    Real.1 <= Real.1
    unit_disk_contains_eq(Complex.i)
    unit_disk.contains(Complex.i)
}

/// Every root of unity lies in the closed unit disk.
theorem unit_disk_contains_omega(n: Nat) {
    n >= Nat.1 implies unit_disk.contains(omega(n))
} by {
    if n >= Nat.1 {
        unit_omega(n)
        n >= Nat.1 implies is_unit(omega(n))
        is_unit(omega(n))
        unit_circle_in_unit_disk(omega(n))
        is_unit(omega(n)) implies unit_disk.contains(omega(n))
        unit_disk.contains(omega(n))
    }
}

/// Zero lies in the open unit disk.
theorem open_unit_disk_contains_zero {
    open_disk(Complex.0, Real.1).contains(Complex.0)
} by {
    open_disk_contains_eq(Complex.0, Real.1, Complex.0)
    open_disk(Complex.0, Real.1).contains(Complex.0) = ((Complex.0 - Complex.0).modulus < Real.1)
    modulus_sub_self_zero(Complex.0)
    (Complex.0 - Complex.0).modulus = Real.0
    Real.0 < Real.1
    open_disk(Complex.0, Real.1).contains(Complex.0)
}

