/// The fundamental theorem of algebra for general polynomials: every
/// nonconstant polynomial with complex coefficients has a complex root.
///
/// This file develops the analytic infrastructure for the classical
/// minimum-modulus proof, factored into standalone lemmas:
///
///   - finite sums and the sum expansion of polynomial evaluation;
///   - modulus bounds for sums (triangle and reverse triangle);
///   - boundedness of a polynomial on a closed disk, and the growth of
///     `|p(z)|` as `|z|` tends to infinity;
///   - the perturbation lemma (D'Alembert's lemma): if `p(z0) != 0` and `p`
///     is nonconstant, some nearby point evaluates to strictly smaller
///     modulus;
///   - epsilon-delta continuity of polynomial evaluation;
///   - the extreme value theorem on a closed disk (recorded, not yet
///     proved: it needs compactness of closed disks in the complex plane,
///     which the library does not yet provide);
///   - the final assembly of the fundamental theorem of algebra.
///
/// The proof follows the standard elementary route: `|p|` is continuous
/// and grows to infinity, so it attains a global minimum at some `z0`; if
/// `p(z0) != 0`, the perturbation lemma produces a point of strictly
/// smaller modulus, contradicting minimality; hence `p(z0) = 0`.
///
/// The perturbation argument needs no roots of arbitrary complex numbers:
/// the direction of the perturbation is chosen among the first four powers
/// of `omega(4k)` (the primitive `4k`-th root of unity), whose `k`-th
/// powers are the fourth roots of unity `1, i, -1, -i`, and one of the
/// four corresponding real parts is always negative.

from nat import Nat, alt_induction, zero_or_suc, lt_or_lte, lt_suc, lt_imp_lte_suc,
    lt_and_lte, lt_not_symm, lt_cancel_suc, lte_imp_not_lt, lte_suc_suc, add_sub, lte_cancel_suc,
    one_pow, suc_sub_one, add_imp_sub_left, add_comm, add_assoc, from_nat_mul, from_nat
from order import lt_imp_lte, lte_refl, lte_trans, lte_antisymm, lt_lte_trans, lte_lt_trans,
    min_lte_left, min_lte_right
from real import Real, two, two_nonzero, one_half_positive, one_half_plus_one_half, pi,
    mul_nonneg, square_nonneg, add_lte_add, lte_add_right, lte_add_left,
    lt_trans, lt_imp_minus_pos, mul_le_mul_of_nonneg_right, mul_le_mul_of_nonneg_left,
    lte_self, sub_cancels, lte_or_gte, not_lt_imp_gte, gte_imp_not_lt, lt_add_right,
    neg_neg, neg_distrib, div_mul_cancel_left,
    mul_div_cancel, div_cancel_common, real_mul_comm,
    from_nat_suc_pos_real, lte_abs, abs_gte_zero, abs_neg,
    min_pos_pos, lte_min_of_bounds, add_neg_eq_zero, mul_sub_distrib_right, mul_sub_distrib_left,
    mul_pos_pos, mul_lt_mul_of_pos_right
from data.nat.nat_range_sum import range_sum, range_sum_suc, range_sum_zero, range_sum_congr,
    range_sum_add, range_sum_one, range_sum_split, shift_fn, range_sum_zero_fn, zero_fn
from complex.complex import Complex, eq_by_components, re_mul, im_mul, mul_inverse,
    mul_one_right, mul_one_left, mul_assoc, mul_comm, distrib, left_distrib,
    add_zero_left, add_zero_right, add_neg_cancel,
    neg_re, neg_im, i_squared, conj_mul, conj_conj, conj_add, conj_from_real,
    re_add, im_add, re_neg, im_neg, from_real_one, from_real_zero, real_add_lifts,
    real_mul_lifts, mul_zero_right, mul_zero_left, re_from_real, re_conj, conj_zero
from complex.complex_abs_deep import modulus_pos
from complex.complex_algebra_deep import complex_eq_new_components
from complex.complex_region_deep import real_pow_le_one
from complex.fundamental_theorem_algebra import one_half_mul_two, two_mul_eq_add_self_real, real_four
from complex.roots_of_unity import omega, complex_i_mul_real_mul, complex_exp_pow, complex_exp_add,
    complex_i_mul_real_add, euler_identity, nat_ge_one_ne_zero, from_nat_two_real, complex_pow_pow
from complex.complex_roots_deep import omega_modulus, complex_pow_mul_distrib
from complex.complex_algebra_deep import from_real_pow
from complex.complex_abs import complex_modulus, modulus_squared, modulus_nonneg, square_le_square_of_nonneg,
    modulus_mul, modulus_triangle, modulus_reverse_triangle, modulus_neg, modulus_from_real_nonneg,
    re_le_modulus, abs_squared_eq, nonneg_eq_of_squares_eq, modulus_eq_zero,
    re_abs_le_modulus, abs_squared_add, modulus_of_zero
from complex.complex_exp import complex_pow_suc, complex_modulus_pow, real_pow_suc, pow_zero,
    real_one_positive, real_two_positive, complex_exp, complex_i_mul_real
from algebra.field.field import Field, field_mul_nonzero, field_square_nonzero
from algebra.ring.ring import mul_sub_left, mul_sub_right, mul_neg_left, mul_neg_right, mul_neg_neg
from algebra.add_group import inverse_add, inverse_inverse, inverse_left, left_cancel, right_cancel
from polynomial import Polynomial, polynomial_eval, polynomial_eval_constant,
    polynomial_eval_add, polynomial_eval_mul, polynomial_monomial, polynomial_constant,
    polynomial_support_bounded_by, polynomial_support_bound_exists,
    polynomial_support_bounded_by_monotone, polynomial_support_bounded_by_apply,
    polynomial_eval_eq_eval_bound_of_support_bounded,
    polynomial_eval_bound, polynomial_eval_bound_eq_coeff_eval,
    coeff_eval, coeff_tail, coeff_zero_from, coeff_zero_from_at, coeff_zero_at,
    coeff_eval_add, coeff_quotient, polynomial_quotient,
    polynomial_quotient_support_bounded_by, polynomial_quotient_support_bounded_by_pred,
    polynomial_remainder_theorem_bundled_bound,
    polynomial_eval_zero, polynomial_ext_pointwise, polynomial_add_coeff,
    polynomial_eq_zero_of_coeff_zero
from semiring import Semiring
from algebra.well_founded import exists_least_nat_witness, is_least_nat_witness

numerals Nat

/// The summand `c(i) * x^i` of the monomial expansion of polynomial
/// evaluation at `x`, for a coefficient function `c`.
define coeff_pow_term(c: Nat -> Complex, x: Complex, i: Nat) -> Complex {
    c(i) * x.pow(i)
}

/// The summand `|c(i)| * r^i` bounding the modulus of a term of degree `i`.
define coeff_pow_bound_term(c: Nat -> Complex, r: Real, i: Nat) -> Real {
    c(i).modulus * r.pow(i)
}

/// Pointwise comparison of two real summands over a bounded range.
define pointwise_lte(f: Nat -> Real, g: Nat -> Real, n: Nat) -> Bool {
    forall(i: Nat) {
        i < n implies f(i) <= g(i)
    }
}

/// A range sum of real summands respects pointwise comparison.
theorem range_sum_le_real(f: Nat -> Real, g: Nat -> Real, n: Nat) {
    pointwise_lte(f, g, n) implies range_sum(f, n) <= range_sum(g, n)
} by {
    define p(k: Nat) -> Bool {
        pointwise_lte(f, g, k) implies range_sum(f, k) <= range_sum(g, k)
    }
    range_sum_zero(f)
    range_sum_zero(g)
    lte_self(Real.0)
    Real.0 <= Real.0
    range_sum(f, Nat.0) = Real.0
    range_sum(g, Nat.0) = Real.0
    range_sum(f, Nat.0) <= range_sum(g, Nat.0)
    p(Nat.0)
    forall(k: Nat) {
        if p(k) {
            if pointwise_lte(f, g, k.suc) {
                pointwise_lte(f, g, k.suc) = forall(i: Nat) {
                    i < k.suc implies f(i) <= g(i)
                }
                forall(i: Nat) {
                    if i < k {
                        i < k.suc
                        i < k.suc implies f(i) <= g(i)
                        f(i) <= g(i)
                    }
                }
                pointwise_lte(f, g, k) = forall(i: Nat) {
                    i < k implies f(i) <= g(i)
                }
                forall(i: Nat) {
                    if i < k {
                        i < k implies f(i) <= g(i)
                        f(i) <= g(i)
                    }
                }
                pointwise_lte(f, g, k)
                p(k) = (pointwise_lte(f, g, k) implies range_sum(f, k) <= range_sum(g, k))
                range_sum(f, k) <= range_sum(g, k)
                lt_suc(k)
                k < k.suc
                k < k.suc implies f(k) <= g(k)
                f(k) <= g(k)
                range_sum_suc(f, k)
                range_sum_suc(g, k)
                range_sum(f, k.suc) = range_sum(f, k) + f(k)
                range_sum(g, k.suc) = range_sum(g, k) + g(k)
                add_lte_add(range_sum(f, k), range_sum(g, k), f(k), g(k))
                range_sum(f, k) + f(k) <= range_sum(g, k) + g(k)
                range_sum(f, k.suc) <= range_sum(g, k.suc)
            }
            p(k.suc)
        }
    }
    p(Nat.0) and forall(k: Nat) {
        p(k) implies p(k.suc)
    }
    alt_induction(p)
    p(n)
}

/// A scalar distributes over a range sum on the left.
theorem range_sum_scale_left(x: Complex, f: Nat -> Complex, n: Nat) {
    range_sum(function(i: Nat) { x * f(i) }, n) = x * range_sum(f, n)
} by {
    define p(k: Nat) -> Bool {
        range_sum(function(i: Nat) { x * f(i) }, k) = x * range_sum(f, k)
    }
    range_sum_zero(f)
    range_sum_zero(function(i: Nat) { x * f(i) })
    mul_zero_right(x)
    x * range_sum(f, Nat.0) = Complex.0
    p(Nat.0)
    forall(k: Nat) {
        if p(k) {
            range_sum_suc(function(i: Nat) { x * f(i) }, k)
            range_sum(function(i: Nat) { x * f(i) }, k.suc) = range_sum(function(i: Nat) { x * f(i) }, k) + x * f(k)
            range_sum_suc(f, k)
            x * range_sum(f, k.suc) = x * (range_sum(f, k) + f(k))
            distrib(x, range_sum(f, k), f(k))
            x * (range_sum(f, k) + f(k)) = x * range_sum(f, k) + x * f(k)
            range_sum(function(i: Nat) { x * f(i) }, k) = x * range_sum(f, k)
            range_sum(function(i: Nat) { x * f(i) }, k.suc) = x * range_sum(f, k) + x * f(k)
            x * range_sum(f, k) + x * f(k) = x * f(k) + x * range_sum(f, k)
            range_sum(function(i: Nat) { x * f(i) }, k.suc) = x * f(k) + x * range_sum(f, k)
            p(k.suc)
        }
    }
    p(Nat.0) and forall(k: Nat) {
        p(k) implies p(k.suc)
    }
    alt_induction(p)
    p(n)
}

/// `x * (c * x^i) = c * x^(i+1)`.
theorem scale_pow_step(c: Complex, x: Complex, i: Nat) {
    x * (c * x.pow(i)) = c * x.pow(i.suc)
} by {
    complex_pow_suc(x, i)
    x.pow(i.suc) = x * x.pow(i)
    mul_assoc(x, c, x.pow(i))
    x * (c * x.pow(i)) = (x * c) * x.pow(i)
    mul_comm(x, c)
    x * c = c * x
    (x * c) * x.pow(i) = (c * x) * x.pow(i)
    mul_assoc(c, x, x.pow(i))
    (c * x) * x.pow(i) = c * (x * x.pow(i))
    c * (x * x.pow(i)) = c * x.pow(i.suc)
    x * (c * x.pow(i)) = c * x.pow(i.suc)
}

/// Horner evaluation at zero is zero.
theorem coeff_eval_nat_zero[R: Semiring](c: Nat -> R, x: R) {
    coeff_eval(c, x, Nat.0) = R.0
}

/// Horner evaluation expands into the sum of monomial terms:
/// `coeff_eval(c, x, n) = sum_{i < n} c(i) * x^i`.
theorem coeff_eval_eq_range_sum(c: Nat -> Complex, x: Complex, n: Nat) {
    coeff_eval(c, x, n) = range_sum(coeff_pow_term(c, x), n)
} by {
    define p(k: Nat) -> Bool {
        forall(d: Nat -> Complex) {
            coeff_eval(d, x, k) = range_sum(coeff_pow_term(d, x), k)
        }
    }
    coeff_eval_nat_zero(c, x)
    coeff_eval(c, x, Nat.0) = Complex.0
    range_sum_zero(coeff_pow_term(c, x))
    range_sum(coeff_pow_term(c, x), Nat.0) = Complex.0
    forall(d: Nat -> Complex) {
        coeff_eval_nat_zero(d, x)
        coeff_eval(d, x, Nat.0) = Complex.0
        range_sum_zero(coeff_pow_term(d, x))
        range_sum(coeff_pow_term(d, x), Nat.0) = Complex.0
        coeff_eval(d, x, Nat.0) = range_sum(coeff_pow_term(d, x), Nat.0)
    }
    p(Nat.0)
    forall(k: Nat) {
        if p(k) {
            forall(d: Nat -> Complex) {
                coeff_eval(d, x, k.suc) = d(Nat.0) + x * coeff_eval(coeff_tail(d), x, k)
                p(k) = forall(e: Nat -> Complex) {
                    coeff_eval(e, x, k) = range_sum(coeff_pow_term(e, x), k)
                }
                coeff_eval(coeff_tail(d), x, k) = range_sum(coeff_pow_term(coeff_tail(d), x), k)
                x * coeff_eval(coeff_tail(d), x, k) = x * range_sum(coeff_pow_term(coeff_tail(d), x), k)
                range_sum_scale_left(x, coeff_pow_term(coeff_tail(d), x), k)
                range_sum(function(i: Nat) { x * coeff_pow_term(coeff_tail(d), x, i) }, k) = x * range_sum(coeff_pow_term(coeff_tail(d), x), k)
                forall(i: Nat) {
                    if i < k {
                        coeff_pow_term(coeff_tail(d), x, i) = coeff_tail(d, i) * x.pow(i)
                        coeff_tail(d, i) = d(i.suc)
                        x * coeff_pow_term(coeff_tail(d), x, i) = x * (d(i.suc) * x.pow(i))
                        scale_pow_step(d(i.suc), x, i)
                        x * (d(i.suc) * x.pow(i)) = d(i.suc) * x.pow(i.suc)
                        x * coeff_pow_term(coeff_tail(d), x, i) = d(i.suc) * x.pow(i.suc)
                        coeff_pow_term(d, x, i.suc) = d(i.suc) * x.pow(i.suc)
                        x * coeff_pow_term(coeff_tail(d), x, i) = coeff_pow_term(d, x, i.suc)
                    }
                }
                range_sum_congr(function(i: Nat) { x * coeff_pow_term(coeff_tail(d), x, i) },
                    function(i: Nat) { coeff_pow_term(d, x, i.suc) }, k)
                range_sum(function(i: Nat) { x * coeff_pow_term(coeff_tail(d), x, i) }, k) = range_sum(function(i: Nat) { coeff_pow_term(d, x, i.suc) }, k)
                x * coeff_eval(coeff_tail(d), x, k) = range_sum(function(i: Nat) { coeff_pow_term(d, x, i.suc) }, k)
                coeff_eval(d, x, k.suc) = d(Nat.0) + range_sum(function(i: Nat) { coeff_pow_term(d, x, i.suc) }, k)
                range_sum_split(coeff_pow_term(d, x), Nat.1, k)
                range_sum(coeff_pow_term(d, x), Nat.1 + k) = range_sum(coeff_pow_term(d, x), Nat.1) + range_sum(shift_fn(coeff_pow_term(d, x), Nat.1), k)
                range_sum_one(coeff_pow_term(d, x))
                range_sum(coeff_pow_term(d, x), Nat.1) = coeff_pow_term(d, x, Nat.0)
                coeff_pow_term(d, x, Nat.0) = d(Nat.0) * x.pow(Nat.0)
                pow_zero(x)
                x.pow(Nat.0) = Complex.1
                mul_one_right(d(Nat.0))
                d(Nat.0) * Complex.1 = d(Nat.0)
                range_sum(coeff_pow_term(d, x), Nat.1) = d(Nat.0)
                Nat.1 + k = k.suc
                range_sum(coeff_pow_term(d, x), k.suc) = d(Nat.0) + range_sum(shift_fn(coeff_pow_term(d, x), Nat.1), k)
                forall(i: Nat) {
                    if i < k {
                        shift_fn(coeff_pow_term(d, x), Nat.1, i) = coeff_pow_term(d, x, Nat.1 + i)
                        Nat.1 + i = i.suc
                        shift_fn(coeff_pow_term(d, x), Nat.1, i) = coeff_pow_term(d, x, i.suc)
                    }
                }
                range_sum_congr(shift_fn(coeff_pow_term(d, x), Nat.1),
                    function(i: Nat) { coeff_pow_term(d, x, i.suc) }, k)
                range_sum(shift_fn(coeff_pow_term(d, x), Nat.1), k) = range_sum(function(i: Nat) { coeff_pow_term(d, x, i.suc) }, k)
                range_sum(coeff_pow_term(d, x), k.suc) = d(Nat.0) + range_sum(function(i: Nat) { coeff_pow_term(d, x, i.suc) }, k)
                coeff_eval(d, x, k.suc) = range_sum(coeff_pow_term(d, x), k.suc)
            }
            p(k.suc)
        }
    }
    p(Nat.0) and forall(k: Nat) {
        p(k) implies p(k.suc)
    }
    alt_induction(p)
    p(n)
    p(n) = forall(d: Nat -> Complex) {
        coeff_eval(d, x, n) = range_sum(coeff_pow_term(d, x), n)
    }
    coeff_eval(c, x, n) = range_sum(coeff_pow_term(c, x), n)
}

/// Polynomial evaluation expands into the sum of monomial terms at any
/// support bound: `p(x) = sum_{i < n} p.coeff(i) * x^i`.
theorem polynomial_eval_eq_range_sum(p: Polynomial[Complex], x: Complex, n: Nat) {
    polynomial_support_bounded_by(p, n) implies polynomial_eval(p, x) = range_sum(coeff_pow_term(p.coeff, x), n)
} by {
    if polynomial_support_bounded_by(p, n) {
        polynomial_eval_eq_eval_bound_of_support_bounded(p, x, n)
        polynomial_eval(p, x) = polynomial_eval_bound(p, x, n)
        polynomial_eval_bound_eq_coeff_eval(p, x, n)
        polynomial_eval_bound(p, x, n) = coeff_eval(p.coeff, x, n)
        coeff_eval_eq_range_sum(p.coeff, x, n)
        coeff_eval(p.coeff, x, n) = range_sum(coeff_pow_term(p.coeff, x), n)
        polynomial_eval(p, x) = range_sum(coeff_pow_term(p.coeff, x), n)
    }
}

// ---------------------------------------------------------------------------
// Section 3.  Modulus bounds for sums, and real power monotonicity.
// ---------------------------------------------------------------------------

/// A nonnegative base raised to a power stays nonnegative.
theorem real_pow_nonneg(x: Real, n: Nat) {
    x >= Real.0 implies x.pow(n) >= Real.0
} by {
    define p(k: Nat) -> Bool {
        x >= Real.0 implies x.pow(k) >= Real.0
    }
    pow_zero(x)
    x.pow(Nat.0) = Real.1
    Real.1 >= Real.0
    p(Nat.0)
    forall(k: Nat) {
        if p(k) {
            if x >= Real.0 {
                x.pow(k) >= Real.0
                real_pow_suc(x, k)
                x.pow(k.suc) = x * x.pow(k)
                mul_nonneg(x, x.pow(k))
                x >= Real.0 and x.pow(k) >= Real.0 implies x * x.pow(k) >= Real.0
                x.pow(k.suc) >= Real.0
            }
            p(k.suc)
        }
    }
    p(Nat.0) and forall(k: Nat) {
        p(k) implies p(k.suc)
    }
    alt_induction(p)
    p(n)
}

/// Powers are monotone in the base for nonnegative bases:
/// `0 <= x <= y` implies `x^n <= y^n`.
theorem real_pow_le_of_lte_base(x: Real, y: Real, n: Nat) {
    x >= Real.0 and x <= y implies x.pow(n) <= y.pow(n)
} by {
    define p(k: Nat) -> Bool {
        x >= Real.0 and x <= y implies x.pow(k) <= y.pow(k)
    }
    pow_zero(x)
    pow_zero(y)
    x.pow(Nat.0) = Real.1
    y.pow(Nat.0) = Real.1
    lte_self(Real.1)
    Real.1 <= Real.1
    p(Nat.0)
    forall(k: Nat) {
        if p(k) {
            if x >= Real.0 and x <= y {
                x.pow(k) <= y.pow(k)
                real_pow_nonneg(x, k)
                x.pow(k) >= Real.0
                lte_trans(Real.0, x, y)
                Real.0 <= y
                real_pow_nonneg(y, k)
                y.pow(k) >= Real.0
                real_pow_suc(x, k)
                real_pow_suc(y, k)
                x.pow(k.suc) = x * x.pow(k)
                y.pow(k.suc) = y * y.pow(k)
                real_mul_comm(x, x.pow(k))
                x * x.pow(k) = x.pow(k) * x
                x.pow(k.suc) = x.pow(k) * x
                real_mul_comm(y, y.pow(k))
                y * y.pow(k) = y.pow(k) * y
                y.pow(k.suc) = y.pow(k) * y
                mul_le_mul_of_nonneg_right(x.pow(k), y.pow(k), x)
                x.pow(k) <= y.pow(k) and x >= Real.0 implies x.pow(k) * x <= y.pow(k) * x
                x.pow(k) * x <= y.pow(k) * x
                x.pow(k.suc) <= y.pow(k) * x
                mul_le_mul_of_nonneg_left(x, y, y.pow(k))
                x <= y and y.pow(k) >= Real.0 implies y.pow(k) * x <= y.pow(k) * y
                y.pow(k) * x <= y.pow(k) * y
                lte_trans(x.pow(k.suc), y.pow(k) * x, y.pow(k) * y)
                x.pow(k.suc) <= y.pow(k) * y
                y.pow(k) * y = y.pow(k.suc)
                x.pow(k.suc) <= y.pow(k.suc)
            }
            p(k.suc)
        }
    }
    p(Nat.0) and forall(k: Nat) {
        p(k) implies p(k.suc)
    }
    alt_induction(p)
    p(n)
}

/// Powers of a base at least one are at least one.
theorem real_pow_ge_one(x: Real, n: Nat) {
    x >= Real.1 implies x.pow(n) >= Real.1
} by {
    define p(k: Nat) -> Bool {
        x >= Real.1 implies x.pow(k) >= Real.1
    }
    pow_zero(x)
    x.pow(Nat.0) = Real.1
    lte_self(Real.1)
    Real.1 <= Real.1
    p(Nat.0)
    forall(k: Nat) {
        if p(k) {
            if x >= Real.1 {
                x.pow(k) >= Real.1
                real_pow_suc(x, k)
                x.pow(k.suc) = x * x.pow(k)
                real_one_positive
                Real.1 > Real.0
                lt_imp_lte(Real.0, Real.1)
                Real.0 <= Real.1
                lte_trans(Real.0, Real.1, x)
                Real.0 <= x
                real_pow_nonneg(x, k)
                x.pow(k) >= Real.0
                mul_le_mul_of_nonneg_left(Real.1, x, x.pow(k))
                Real.1 <= x and x.pow(k) >= Real.0 implies x.pow(k) * Real.1 <= x.pow(k) * x
                x.pow(k) >= Real.1
                Real.1 <= x.pow(k)
                x.pow(k) * Real.1 <= x.pow(k) * x
                x.pow(k) * Real.1 = x.pow(k)
                x.pow(k) <= x.pow(k) * x
                lte_trans(Real.1, x.pow(k), x.pow(k) * x)
                Real.1 <= x.pow(k) * x
                x.pow(k.suc) >= Real.1
            }
            p(k.suc)
        }
    }
    p(Nat.0) and forall(k: Nat) {
        p(k) implies p(k.suc)
    }
    alt_induction(p)
    p(n)
}

/// Powers are monotone in the exponent for bases at least one:
/// `x >= 1` and `i <= j` imply `x^i <= x^j`.
theorem real_pow_le_of_lte_exp(x: Real, i: Nat, j: Nat) {
    x >= Real.1 and i <= j implies x.pow(i) <= x.pow(j)
} by {
    define p(m: Nat) -> Bool {
        forall(t: Nat) {
            t <= m implies x.pow(t) <= x.pow(m)
        }
    }
    pow_zero(x)
    x.pow(Nat.0) = Real.1
    forall(t: Nat) {
        if t <= Nat.0 {
            lte_antisymm(t, Nat.0)
            t = Nat.0
            lte_self(x.pow(t))
            x.pow(t) <= x.pow(t)
            x.pow(t) <= x.pow(Nat.0)
        }
    }
    p(Nat.0)
    forall(m: Nat) {
        if p(m) {
            forall(t: Nat) {
                if t <= m.suc {
                    lt_or_lte(t, m.suc)
                    if t <= m {
                        x.pow(t) <= x.pow(m)
                        real_pow_suc(x, m)
                        x.pow(m.suc) = x * x.pow(m)
                        real_pow_ge_one(x, m)
                        x.pow(m) >= Real.1
                        real_one_positive
                        Real.1 > Real.0
                        lt_imp_lte(Real.0, Real.1)
                        Real.0 <= Real.1
                        lte_trans(Real.0, Real.1, x)
                        Real.0 <= x
                        real_pow_nonneg(x, m)
                        x.pow(m) >= Real.0
                        mul_le_mul_of_nonneg_left(Real.1, x, x.pow(m))
                        Real.1 <= x and x.pow(m) >= Real.0 implies x.pow(m) * Real.1 <= x.pow(m) * x
                        x.pow(m) * Real.1 <= x.pow(m) * x
                        x.pow(m) * Real.1 = x.pow(m)
                        x.pow(m) <= x.pow(m) * x
                        lte_trans(x.pow(t), x.pow(m), x.pow(m) * x)
                        x.pow(t) <= x.pow(m.suc)
                    } else {
                        not t <= m
                        not t < m.suc
                        lt_or_lte(t, m.suc)
                        m.suc <= t
                        lte_antisymm(m.suc, t)
                        t = m.suc
                        lte_self(x.pow(t))
                        x.pow(t) <= x.pow(t)
                        x.pow(t) <= x.pow(m.suc)
                    }
                }
            }
            p(m.suc)
        }
    }
    p(Nat.0) and forall(m: Nat) {
        p(m) implies p(m.suc)
    }
    alt_induction(p)
    p(j)
}

/// The modulus of a range sum is at most the range sum of the moduli.
theorem modulus_range_sum_le(f: Nat -> Complex, n: Nat) {
    (range_sum(f, n)).modulus <= range_sum(function(i: Nat) { f(i).modulus }, n)
} by {
    define p(k: Nat) -> Bool {
        (range_sum(f, k)).modulus <= range_sum(function(i: Nat) { f(i).modulus }, k)
    }
    range_sum_zero(f)
    range_sum_zero(function(i: Nat) { f(i).modulus })
    (range_sum(f, Nat.0)).modulus = Real.0
    lte_self(Real.0)
    Real.0 <= Real.0
    range_sum(function(i: Nat) { f(i).modulus }, Nat.0) = Real.0
    (range_sum(f, Nat.0)).modulus <= range_sum(function(i: Nat) { f(i).modulus }, Nat.0)
    p(Nat.0)
    forall(k: Nat) {
        if p(k) {
            range_sum_suc(f, k)
            range_sum_suc(function(i: Nat) { f(i).modulus }, k)
            range_sum(f, k.suc) = range_sum(f, k) + f(k)
            range_sum(function(i: Nat) { f(i).modulus }, k.suc) = range_sum(function(i: Nat) { f(i).modulus }, k) + f(k).modulus
            modulus_triangle(range_sum(f, k), f(k))
            (range_sum(f, k) + f(k)).modulus <= (range_sum(f, k)).modulus + f(k).modulus
            p(k) = ((range_sum(f, k)).modulus <= range_sum(function(i: Nat) { f(i).modulus }, k))
            (range_sum(f, k)).modulus + f(k).modulus <= range_sum(function(i: Nat) { f(i).modulus }, k) + f(k).modulus
            add_lte_add((range_sum(f, k)).modulus,
                range_sum(function(i: Nat) { f(i).modulus }, k),
                f(k).modulus, f(k).modulus)
            lte_trans((range_sum(f, k) + f(k)).modulus,
                (range_sum(f, k)).modulus + f(k).modulus,
                range_sum(function(i: Nat) { f(i).modulus }, k) + f(k).modulus)
            (range_sum(f, k.suc)).modulus <= range_sum(function(i: Nat) { f(i).modulus }, k.suc)
            p(k.suc)
        }
    }
    p(Nat.0) and forall(k: Nat) {
        p(k) implies p(k.suc)
    }
    alt_induction(p)
    p(n)
}

/// The modulus of a difference is at least the difference of the moduli:
/// `|a - b| >= |a| - |b|`.
theorem modulus_sub_lower(a: Complex, b: Complex) {
    (a - b).modulus >= a.modulus - b.modulus
} by {
    modulus_reverse_triangle(a, b)
    (a.modulus - b.modulus).abs <= (a - b).modulus
    lte_abs(a.modulus - b.modulus)
    a.modulus - b.modulus <= (a.modulus - b.modulus).abs
    lte_trans(a.modulus - b.modulus, (a.modulus - b.modulus).abs, (a - b).modulus)
    a.modulus - b.modulus <= (a - b).modulus
}

/// The modulus of a sum is at least the difference of the moduli:
/// `|a + b| >= |a| - |b|`.
theorem modulus_add_lower(a: Complex, b: Complex) {
    (a + b).modulus >= a.modulus - b.modulus
} by {
    a + b = a - -b
    (a + b).modulus = (a - -b).modulus
    modulus_sub_lower(a, -b)
    (a - -b).modulus >= a.modulus - (-b).modulus
    modulus_neg(b)
    (-b).modulus = b.modulus
    a.modulus - (-b).modulus = a.modulus - b.modulus
    (a + b).modulus >= a.modulus - b.modulus
}

/// The modulus of the product of a term and a power is the product of the
/// moduli: `|c * z^i| = |c| * |z|^i`.
theorem coeff_pow_term_modulus(c: Complex, z: Complex, i: Nat) {
    (c * z.pow(i)).modulus = c.modulus * z.modulus.pow(i)
} by {
    modulus_mul(c, z.pow(i))
    (c * z.pow(i)).modulus = c.modulus * (z.pow(i)).modulus
    complex_modulus_pow(z, i)
    z.pow(i).modulus = z.modulus.pow(i)
    (c * z.pow(i)).modulus = c.modulus * z.modulus.pow(i)
}

/// A modulus below a radius bounds every power: `|z| <= r` and `r >= 0`
/// imply `|z|^i <= r^i`.
theorem modulus_pow_le_radius(z: Complex, r: Real, i: Nat) {
    z.modulus >= Real.0 and z.modulus <= r implies z.modulus.pow(i) <= r.pow(i)
} by {
    real_pow_le_of_lte_base(z.modulus, r, i)
    z.modulus >= Real.0 and z.modulus <= r implies z.modulus.pow(i) <= r.pow(i)
    z.modulus >= Real.0 and z.modulus <= r
    z.modulus.pow(i) <= r.pow(i)
}

/// A natural strictly below `n >= 1` is at most `n - 1`.
theorem nat_lte_pred_of_lt(i: Nat, n: Nat) {
    i < n and n >= Nat.1 implies i <= n - 1
} by {
    if i < n and n >= Nat.1 {
        lt_imp_lte_suc(i, n)
        i.suc <= n
        zero_or_suc(n)
        n >= Nat.1
        n != Nat.0
        let m: Nat satisfy {
            n = m.suc
        }
        n = m.suc
        n - 1 = m.suc - 1
        m.suc - 1 = m
        n - 1 = m
        i.suc <= m.suc
        lte_cancel_suc(i, m)
        i <= m
        i <= n - 1
    }
}

/// Powers with exponent below `n` are bounded by the `(n-1)`-st power for a
/// base at least one: `x >= 1` and `i < n` imply `x^i <= x^(n-1)`.
theorem real_pow_le_pred_pow(x: Real, i: Nat, n: Nat) {
    x >= Real.1 and i < n and n >= Nat.1 implies x.pow(i) <= x.pow(n - 1)
} by {
    if x >= Real.1 and i < n and n >= Nat.1 {
        nat_lte_pred_of_lt(i, n)
        i <= n - 1
        real_pow_le_of_lte_exp(x, i, n - 1)
        x >= Real.1 and i <= n - 1 implies x.pow(i) <= x.pow(n - 1)
        x.pow(i) <= x.pow(n - 1)
    }
}

/// Summing a scalar bound over a range distributes the scalar out.
theorem range_sum_scale_right(c: Real, f: Nat -> Real, n: Nat) {
    range_sum(function(i: Nat) { c * f(i) }, n) = c * range_sum(f, n)
} by {
    define p(k: Nat) -> Bool {
        range_sum(function(i: Nat) { c * f(i) }, k) = c * range_sum(f, k)
    }
    range_sum_zero(f)
    range_sum_zero(function(i: Nat) { c * f(i) })
    c * Real.0 = Real.0
    c * range_sum(f, Nat.0) = Real.0
    p(Nat.0)
    forall(k: Nat) {
        if p(k) {
            range_sum_suc(function(i: Nat) { c * f(i) }, k)
            range_sum(function(i: Nat) { c * f(i) }, k.suc) = range_sum(function(i: Nat) { c * f(i) }, k) + c * f(k)
            range_sum_suc(f, k)
            c * range_sum(f, k.suc) = c * (range_sum(f, k) + f(k))
            c * (range_sum(f, k) + f(k)) = c * range_sum(f, k) + c * f(k)
            range_sum(function(i: Nat) { c * f(i) }, k) = c * range_sum(f, k)
            range_sum(function(i: Nat) { c * f(i) }, k.suc) = c * range_sum(f, k) + c * f(k)
            c * range_sum(f, k) + c * f(k) = c * f(k) + c * range_sum(f, k)
            range_sum(function(i: Nat) { c * f(i) }, k.suc) = c * f(k) + c * range_sum(f, k)
            c * f(k) + c * range_sum(f, k) = c * range_sum(f, k.suc)
            range_sum(function(i: Nat) { c * f(i) }, k.suc) = c * range_sum(f, k.suc)
            p(k.suc)
        }
    }
    p(Nat.0) and forall(k: Nat) {
        p(k) implies p(k.suc)
    }
    alt_induction(p)
    p(n)
}

/// A pointwise upper bound by a constant lifts to the range sum:
/// `f(i) <= c` for all `i < n` implies `sum f <= n * c`.
theorem range_sum_le_const_mul(f: Nat -> Real, c: Real, n: Nat) {
    (forall(i: Nat) { i < n implies f(i) <= c }) and c >= Real.0 implies range_sum(f, n) <= range_sum(function(i: Nat) { c }, n)
} by {
    if forall(i: Nat) { i < n implies f(i) <= c } {
        pointwise_lte(f, function(t: Nat) { c }, n) = forall(u: Nat) {
            u < n implies f(u) <= function(t: Nat) { c }(u)
        }
        forall(i: Nat) {
            if i < n {
                f(i) <= c
                function(t: Nat) { c }(i) = c
                f(i) <= function(t: Nat) { c }(i)
            }
        }
        pointwise_lte(f, function(t: Nat) { c }, n)
        range_sum_le_real(f, function(t: Nat) { c }, n)
        range_sum(f, n) <= range_sum(function(t: Nat) { c }, n)
    }
}

// ---------------------------------------------------------------------------
// Section 4.  Boundedness on a disk and growth to infinity.
// ---------------------------------------------------------------------------

/// Negating both sides of a non-strict inequality reverses it.
theorem lte_imp_neg_lte_neg_local(a: Real, b: Real) {
    a <= b implies -b <= -a
} by {
    if a <= b {
        lte_add_right(a, b, -(a + b))
        a + -(a + b) <= b + -(a + b)
        neg_distrib(a, b)
        -(a + b) = -a + -b
        a + -(a + b) = a + (-a + -b)
        a + (-a + -b) = (a + -a) + -b
        add_neg_eq_zero(a)
        a + -a = Real.0
        Real.0 + -b = -b
        (a + -a) + -b = -b
        a + (-a + -b) = -b
        a + -(a + b) = -b
        neg_distrib(b, a)
        -(a + b) = -a + -b
        b + -(a + b) = b + (-a + -b)
        b + (-a + -b) = (b + -a) + -b
        b + -a = -a + b
        sub_cancels(-a, b)
        (-a + b) + -b = -a
        (b + -a) + -b = -a
        b + (-a + -b) = -a
        b + -(a + b) = -a
        -b <= -a
    }
}

/// A range sum of nonnegative real summands is nonnegative.
theorem range_sum_nonneg_real(f: Nat -> Real, n: Nat) {
    (forall(i: Nat) { i < n implies f(i) >= Real.0 }) implies range_sum(f, n) >= Real.0
} by {
    define p(k: Nat) -> Bool {
        (forall(i: Nat) { i < k implies f(i) >= Real.0 }) implies range_sum(f, k) >= Real.0
    }
    range_sum_zero(f)
    range_sum(f, Nat.0) = Real.0
    lte_self(Real.0)
    Real.0 <= Real.0
    p(Nat.0)
    forall(k: Nat) {
        if p(k) {
            if forall(i: Nat) { i < k.suc implies f(i) >= Real.0 } {
                forall(i: Nat) {
                    if i < k {
                        i < k.suc
                        f(i) >= Real.0
                    }
                }
                p(k) = ((forall(i: Nat) { i < k implies f(i) >= Real.0 }) implies range_sum(f, k) >= Real.0)
                range_sum(f, k) >= Real.0
                k < k.suc
                f(k) >= Real.0
                range_sum_suc(f, k)
                range_sum(f, k.suc) = range_sum(f, k) + f(k)
                add_lte_add(Real.0, range_sum(f, k), Real.0, f(k))
                Real.0 <= range_sum(f, k)
                Real.0 <= f(k)
                Real.0 <= range_sum(f, k) and Real.0 <= f(k) implies Real.0 + Real.0 <= range_sum(f, k) + f(k)
                Real.0 + Real.0 <= range_sum(f, k) + f(k)
                Real.0 + Real.0 = Real.0
                Real.0 <= range_sum(f, k) + f(k)
                range_sum(f, k.suc) >= Real.0
            }
            p(k.suc)
        }
    }
    p(Nat.0) and forall(k: Nat) {
        p(k) implies p(k.suc)
    }
    alt_induction(p)
    p(n)
}

/// If `b <= c` then `a - c <= a - b`.
theorem real_sub_lte_lte(a: Real, b: Real, c: Real) {
    b <= c implies a - c <= a - b
} by {
    if b <= c {
        lte_imp_neg_lte_neg_local(b, c)
        -c <= -b
        lte_add_right(-c, -b, a)
        -c + a <= -b + a
        a + -c <= a + -b
        a - c = a + -c
        a - b = a + -b
        a - c <= a - b
    }
}

/// The modulus of a polynomial evaluation on a disk of radius `r` is
/// bounded by the sum of the coefficient moduli times the powers of `r`.
theorem polynomial_eval_disk_bound(p: Polynomial[Complex], z: Complex, r: Real, n: Nat) {
    polynomial_support_bounded_by(p, n) and r >= Real.0 and z.modulus <= r implies polynomial_eval(p, z).modulus <= range_sum(coeff_pow_bound_term(p.coeff, r), n)
} by {
    if polynomial_support_bounded_by(p, n) and r >= Real.0 and z.modulus <= r {
        polynomial_eval_eq_range_sum(p, z, n)
        polynomial_eval(p, z) = range_sum(coeff_pow_term(p.coeff, z), n)
        modulus_range_sum_le(coeff_pow_term(p.coeff, z), n)
        (range_sum(coeff_pow_term(p.coeff, z), n)).modulus <= range_sum(function(i: Nat) { (coeff_pow_term(p.coeff, z, i)).modulus }, n)
        forall(i: Nat) {
            if i < n {
                coeff_pow_term_modulus(p.coeff(i), z, i)
                (p.coeff(i) * z.pow(i)).modulus = p.coeff(i).modulus * z.modulus.pow(i)
                coeff_pow_term(p.coeff, z, i) = p.coeff(i) * z.pow(i)
                (coeff_pow_term(p.coeff, z, i)).modulus = p.coeff(i).modulus * z.modulus.pow(i)
            }
        }
        range_sum_congr(function(i: Nat) { (coeff_pow_term(p.coeff, z, i)).modulus },
            function(i: Nat) { p.coeff(i).modulus * z.modulus.pow(i) }, n)
        range_sum(function(i: Nat) { (coeff_pow_term(p.coeff, z, i)).modulus }, n) = range_sum(function(i: Nat) { p.coeff(i).modulus * z.modulus.pow(i) }, n)
        (range_sum(coeff_pow_term(p.coeff, z), n)).modulus <= range_sum(function(i: Nat) { p.coeff(i).modulus * z.modulus.pow(i) }, n)
        forall(i: Nat) {
            if i < n {
                modulus_pow_le_radius(z, r, i)
                z.modulus.pow(i) <= r.pow(i)
                modulus_nonneg(p.coeff(i))
                p.coeff(i).modulus >= Real.0
                mul_le_mul_of_nonneg_right(z.modulus.pow(i), r.pow(i), p.coeff(i).modulus)
                z.modulus.pow(i) <= r.pow(i) and p.coeff(i).modulus >= Real.0 implies z.modulus.pow(i) * p.coeff(i).modulus <= r.pow(i) * p.coeff(i).modulus
                z.modulus.pow(i) * p.coeff(i).modulus <= r.pow(i) * p.coeff(i).modulus
                real_mul_comm(z.modulus.pow(i), p.coeff(i).modulus)
                z.modulus.pow(i) * p.coeff(i).modulus = p.coeff(i).modulus * z.modulus.pow(i)
                p.coeff(i).modulus * z.modulus.pow(i) <= r.pow(i) * p.coeff(i).modulus
                real_mul_comm(r.pow(i), p.coeff(i).modulus)
                r.pow(i) * p.coeff(i).modulus = p.coeff(i).modulus * r.pow(i)
                p.coeff(i).modulus * z.modulus.pow(i) <= p.coeff(i).modulus * r.pow(i)
            }
        }
        pointwise_lte(function(i: Nat) { p.coeff(i).modulus * z.modulus.pow(i) },
            function(i: Nat) { p.coeff(i).modulus * r.pow(i) }, n) = forall(t: Nat) {
            t < n implies function(i: Nat) { p.coeff(i).modulus * z.modulus.pow(i) }(t) <= function(i: Nat) { p.coeff(i).modulus * r.pow(i) }(t)
        }
        forall(t: Nat) {
            if t < n {
                t < n implies function(i: Nat) { p.coeff(i).modulus * z.modulus.pow(i) }(t) <= function(i: Nat) { p.coeff(i).modulus * r.pow(i) }(t)
                function(i: Nat) { p.coeff(i).modulus * z.modulus.pow(i) }(t) <= function(i: Nat) { p.coeff(i).modulus * r.pow(i) }(t)
            }
        }
        pointwise_lte(function(i: Nat) { p.coeff(i).modulus * z.modulus.pow(i) },
            function(i: Nat) { p.coeff(i).modulus * r.pow(i) }, n)
        range_sum_le_real(function(i: Nat) { p.coeff(i).modulus * z.modulus.pow(i) },
            function(i: Nat) { p.coeff(i).modulus * r.pow(i) }, n)
        range_sum(function(i: Nat) { p.coeff(i).modulus * z.modulus.pow(i) }, n) <= range_sum(function(i: Nat) { p.coeff(i).modulus * r.pow(i) }, n)
        forall(i: Nat) {
            if i < n {
                coeff_pow_bound_term(p.coeff, r, i) = p.coeff(i).modulus * r.pow(i)
            }
        }
        range_sum_congr(function(i: Nat) { p.coeff(i).modulus * r.pow(i) },
            coeff_pow_bound_term(p.coeff, r), n)
        range_sum(function(i: Nat) { p.coeff(i).modulus * r.pow(i) }, n) = range_sum(coeff_pow_bound_term(p.coeff, r), n)
        lte_trans((range_sum(coeff_pow_term(p.coeff, z), n)).modulus,
            range_sum(function(i: Nat) { p.coeff(i).modulus * z.modulus.pow(i) }, n),
            range_sum(coeff_pow_bound_term(p.coeff, r), n))
        (range_sum(coeff_pow_term(p.coeff, z), n)).modulus <= range_sum(coeff_pow_bound_term(p.coeff, r), n)
        polynomial_eval(p, z).modulus <= range_sum(coeff_pow_bound_term(p.coeff, r), n)
    }
}

/// The sum of the coefficient moduli below `n` is a fixed nonnegative
/// constant attached to `p`.
define coefficient_modulus_sum(p: Polynomial[Complex], n: Nat) -> Real {
    range_sum(coeff_pow_bound_term(p.coeff, Real.1), n)
}

/// The tail of the monomial expansion is bounded by `|z|^(n-1)` times the
/// sum of the coefficient moduli below `n`, for `|z| >= 1`.
theorem polynomial_eval_tail_bound(p: Polynomial[Complex], z: Complex, n: Nat) {
    n >= Nat.1 and z.modulus >= Real.1 implies (range_sum(coeff_pow_term(p.coeff, z), n)).modulus <= z.modulus.pow(n - 1) * coefficient_modulus_sum(p, n)
} by {
    if n >= Nat.1 and z.modulus >= Real.1 {
        modulus_range_sum_le(coeff_pow_term(p.coeff, z), n)
        (range_sum(coeff_pow_term(p.coeff, z), n)).modulus <= range_sum(function(i: Nat) { (coeff_pow_term(p.coeff, z, i)).modulus }, n)
        forall(i: Nat) {
            if i < n {
                coeff_pow_term_modulus(p.coeff(i), z, i)
                (p.coeff(i) * z.pow(i)).modulus = p.coeff(i).modulus * z.modulus.pow(i)
                coeff_pow_term(p.coeff, z, i) = p.coeff(i) * z.pow(i)
                (coeff_pow_term(p.coeff, z, i)).modulus = p.coeff(i).modulus * z.modulus.pow(i)
            }
        }
        range_sum_congr(function(i: Nat) { (coeff_pow_term(p.coeff, z, i)).modulus },
            function(i: Nat) { p.coeff(i).modulus * z.modulus.pow(i) }, n)
        range_sum(function(i: Nat) { (coeff_pow_term(p.coeff, z, i)).modulus }, n) = range_sum(function(i: Nat) { p.coeff(i).modulus * z.modulus.pow(i) }, n)
        (range_sum(coeff_pow_term(p.coeff, z), n)).modulus <= range_sum(function(i: Nat) { p.coeff(i).modulus * z.modulus.pow(i) }, n)
        forall(i: Nat) {
            if i < n {
                one_pow[Real](i)
                Real.1.pow(i) = Real.1
                coeff_pow_bound_term(p.coeff, Real.1, i) = p.coeff(i).modulus * Real.1.pow(i)
                coeff_pow_bound_term(p.coeff, Real.1, i) = p.coeff(i).modulus
                p.coeff(i).modulus * Real.1 = p.coeff(i).modulus
                real_pow_le_pred_pow(z.modulus, i, n)
                z.modulus.pow(i) <= z.modulus.pow(n - 1)
                modulus_nonneg(p.coeff(i))
                p.coeff(i).modulus >= Real.0
                mul_le_mul_of_nonneg_right(z.modulus.pow(i), z.modulus.pow(n - 1), p.coeff(i).modulus)
                z.modulus.pow(i) <= z.modulus.pow(n - 1) and p.coeff(i).modulus >= Real.0 implies z.modulus.pow(i) * p.coeff(i).modulus <= z.modulus.pow(n - 1) * p.coeff(i).modulus
                z.modulus.pow(i) * p.coeff(i).modulus <= z.modulus.pow(n - 1) * p.coeff(i).modulus
                real_mul_comm(z.modulus.pow(i), p.coeff(i).modulus)
                z.modulus.pow(i) * p.coeff(i).modulus = p.coeff(i).modulus * z.modulus.pow(i)
                p.coeff(i).modulus * z.modulus.pow(i) <= z.modulus.pow(n - 1) * p.coeff(i).modulus
                real_mul_comm(z.modulus.pow(n - 1), p.coeff(i).modulus)
                z.modulus.pow(n - 1) * p.coeff(i).modulus = p.coeff(i).modulus * z.modulus.pow(n - 1)
                p.coeff(i).modulus * z.modulus.pow(i) <= p.coeff(i).modulus * z.modulus.pow(n - 1)
            }
        }
        pointwise_lte(function(i: Nat) { p.coeff(i).modulus * z.modulus.pow(i) },
            function(i: Nat) { p.coeff(i).modulus * z.modulus.pow(n - 1) }, n) = forall(t: Nat) {
            t < n implies function(i: Nat) { p.coeff(i).modulus * z.modulus.pow(i) }(t) <= function(i: Nat) { p.coeff(i).modulus * z.modulus.pow(n - 1) }(t)
        }
        forall(t: Nat) {
            if t < n {
                t < n implies function(i: Nat) { p.coeff(i).modulus * z.modulus.pow(i) }(t) <= function(i: Nat) { p.coeff(i).modulus * z.modulus.pow(n - 1) }(t)
                function(i: Nat) { p.coeff(i).modulus * z.modulus.pow(i) }(t) <= function(i: Nat) { p.coeff(i).modulus * z.modulus.pow(n - 1) }(t)
            }
        }
        pointwise_lte(function(i: Nat) { p.coeff(i).modulus * z.modulus.pow(i) },
            function(i: Nat) { p.coeff(i).modulus * z.modulus.pow(n - 1) }, n)
        range_sum_le_real(function(i: Nat) { p.coeff(i).modulus * z.modulus.pow(i) },
            function(i: Nat) { p.coeff(i).modulus * z.modulus.pow(n - 1) }, n)
        range_sum(function(i: Nat) { p.coeff(i).modulus * z.modulus.pow(i) }, n) <= range_sum(function(i: Nat) { p.coeff(i).modulus * z.modulus.pow(n - 1) }, n)
        forall(i: Nat) {
            if i < n {
                real_mul_comm(z.modulus.pow(n - 1), coeff_pow_bound_term(p.coeff, Real.1, i))
                z.modulus.pow(n - 1) * coeff_pow_bound_term(p.coeff, Real.1, i) = coeff_pow_bound_term(p.coeff, Real.1, i) * z.modulus.pow(n - 1)
                one_pow[Real](i)
                Real.1.pow(i) = Real.1
                coeff_pow_bound_term(p.coeff, Real.1, i) = p.coeff(i).modulus * Real.1.pow(i)
                p.coeff(i).modulus * Real.1 = p.coeff(i).modulus
                coeff_pow_bound_term(p.coeff, Real.1, i) = p.coeff(i).modulus
                p.coeff(i).modulus * z.modulus.pow(n - 1) = z.modulus.pow(n - 1) * p.coeff(i).modulus
                coeff_pow_bound_term(p.coeff, Real.1, i) * z.modulus.pow(n - 1) = z.modulus.pow(n - 1) * p.coeff(i).modulus
            }
        }
        range_sum_congr(function(i: Nat) { p.coeff(i).modulus * z.modulus.pow(n - 1) },
            function(i: Nat) { coeff_pow_bound_term(p.coeff, Real.1, i) * z.modulus.pow(n - 1) }, n)
        range_sum(function(i: Nat) { p.coeff(i).modulus * z.modulus.pow(n - 1) }, n) = range_sum(function(i: Nat) { coeff_pow_bound_term(p.coeff, Real.1, i) * z.modulus.pow(n - 1) }, n)
        range_sum_scale_right(z.modulus.pow(n - 1), coeff_pow_bound_term(p.coeff, Real.1), n)
        range_sum(function(i: Nat) { z.modulus.pow(n - 1) * coeff_pow_bound_term(p.coeff, Real.1, i) }, n) = z.modulus.pow(n - 1) * range_sum(coeff_pow_bound_term(p.coeff, Real.1), n)
        forall(i: Nat) {
            if i < n {
                real_mul_comm(z.modulus.pow(n - 1), coeff_pow_bound_term(p.coeff, Real.1, i))
                z.modulus.pow(n - 1) * coeff_pow_bound_term(p.coeff, Real.1, i) = coeff_pow_bound_term(p.coeff, Real.1, i) * z.modulus.pow(n - 1)
            }
        }
        range_sum_congr(function(i: Nat) { z.modulus.pow(n - 1) * coeff_pow_bound_term(p.coeff, Real.1, i) },
            function(i: Nat) { coeff_pow_bound_term(p.coeff, Real.1, i) * z.modulus.pow(n - 1) }, n)
        range_sum(function(i: Nat) { z.modulus.pow(n - 1) * coeff_pow_bound_term(p.coeff, Real.1, i) }, n) = range_sum(function(i: Nat) { coeff_pow_bound_term(p.coeff, Real.1, i) * z.modulus.pow(n - 1) }, n)
        range_sum(function(i: Nat) { p.coeff(i).modulus * z.modulus.pow(n - 1) }, n) = z.modulus.pow(n - 1) * range_sum(coeff_pow_bound_term(p.coeff, Real.1), n)
        coefficient_modulus_sum(p, n) = range_sum(coeff_pow_bound_term(p.coeff, Real.1), n)
        lte_trans((range_sum(coeff_pow_term(p.coeff, z), n)).modulus,
            range_sum(function(i: Nat) { p.coeff(i).modulus * z.modulus.pow(i) }, n),
            z.modulus.pow(n - 1) * range_sum(coeff_pow_bound_term(p.coeff, Real.1), n))
        (range_sum(coeff_pow_term(p.coeff, z), n)).modulus <= z.modulus.pow(n - 1) * range_sum(coeff_pow_bound_term(p.coeff, Real.1), n)
        (range_sum(coeff_pow_term(p.coeff, z), n)).modulus <= z.modulus.pow(n - 1) * coefficient_modulus_sum(p, n)
    }
}

/// Every successor of a natural number is at least one.
theorem nat_suc_ge_one(k: Nat) {
    k.suc >= Nat.1
} by {
    lt_or_lte(k, Nat.0)
    if k < Nat.0 {
        false
    }
    Nat.0 <= k
    lte_suc_suc(Nat.0, k)
    Nat.0.suc <= k.suc
    Nat.1 <= k.suc
}

/// A natural number at least one is the successor of its predecessor.
theorem nat_suc_pred_self(n: Nat) {
    n >= Nat.1 implies (n - 1).suc = n
} by {
    if n >= Nat.1 {
        zero_or_suc(n)
        n != Nat.0
        let m: Nat satisfy {
            n = m.suc
        }
        n = m.suc
        n - 1 = m.suc - 1
        m.suc - 1 = m
        n - 1 = m
        (n - 1).suc = m.suc
        (n - 1).suc = n
    }
}

/// The predecessor of a positive natural number is below it.
theorem nat_pred_lt_self(n: Nat) {
    n >= Nat.1 implies n - 1 < n
} by {
    if n >= Nat.1 {
        nat_suc_pred_self(n)
        (n - 1).suc = n
        lt_suc(n - 1)
        n - 1 < (n - 1).suc
        n - 1 < n
    }
}

/// A natural number that is not at least one is zero.
theorem nat_not_ge_one_imp_zero(n: Nat) {
    not n >= Nat.1 implies n = Nat.0
} by {
    if not n >= Nat.1 {
        zero_or_suc(n)
        if n = Nat.0 {
            n = Nat.0
        } else {
            let k: Nat satisfy {
                n = k.suc
            }
            n = k.suc
            nat_suc_ge_one(k)
            k.suc >= Nat.1
            n >= Nat.1
            false
        }
        n = Nat.0
    }
}

/// A range sum whose positive-index summands vanish is its zeroth summand,
/// for a range of length at least one.
theorem range_sum_zero_above_zero(f: Nat -> Complex, n: Nat) {
    n >= Nat.1 and (forall(i: Nat) { i >= Nat.1 and i < n implies f(i) = Complex.0 }) implies range_sum(f, n) = f(Nat.0)
} by {
    define p(k: Nat) -> Bool {
        (forall(i: Nat) { i >= Nat.1 and i < k.suc implies f(i) = Complex.0 }) implies range_sum(f, k.suc) = f(Nat.0)
    }
    range_sum_one(f)
    range_sum(f, Nat.1) = f(Nat.0)
    p(Nat.0)
    forall(k: Nat) {
        if p(k) {
            if forall(i: Nat) { i >= Nat.1 and i < k.suc.suc implies f(i) = Complex.0 } {
                forall(i: Nat) {
                    if i >= Nat.1 and i < k.suc {
                        i < k.suc.suc
                        f(i) = Complex.0
                    }
                }
                p(k) = ((forall(i: Nat) { i >= Nat.1 and i < k.suc implies f(i) = Complex.0 }) implies range_sum(f, k.suc) = f(Nat.0))
                range_sum(f, k.suc) = f(Nat.0)
                nat_suc_ge_one(k)
                k.suc >= Nat.1
                lt_suc(k.suc)
                k.suc < k.suc.suc
                f(k.suc) = Complex.0
                range_sum_suc(f, k.suc)
                range_sum(f, k.suc.suc) = range_sum(f, k.suc) + f(k.suc)
                range_sum(f, k.suc.suc) = f(Nat.0) + Complex.0
                add_zero_right(f(Nat.0))
                f(Nat.0) + Complex.0 = f(Nat.0)
                range_sum(f, k.suc.suc) = f(Nat.0)
            }
            p(k.suc)
        }
    }
    p(Nat.0) and forall(k: Nat) {
        p(k) implies p(k.suc)
    }
    alt_induction(p)
    zero_or_suc(n)
    n = Nat.0 or exists(k: Nat) { n = k.suc }
    if n = Nat.0 {
        false
    }
    let k: Nat satisfy {
        n = k.suc
    }
    p(k)
    n >= Nat.1 and (forall(i: Nat) { i >= Nat.1 and i < n implies f(i) = Complex.0 })
    p(k) = ((forall(i: Nat) { i >= Nat.1 and i < k.suc implies f(i) = Complex.0 }) implies range_sum(f, k.suc) = f(Nat.0))
    n = k.suc
    forall(i: Nat) { i >= Nat.1 and i < k.suc implies f(i) = Complex.0 }
    range_sum(f, k.suc) = f(Nat.0)
    range_sum(f, n) = f(Nat.0)
}

/// A polynomial taking two distinct values has a nonzero coefficient at
/// some positive index.
theorem nonconstant_has_nonzero_high_coeff(p: Polynomial[Complex]) {
    exists(x: Complex, y: Complex) { polynomial_eval(p, x) != polynomial_eval(p, y) } implies exists(j: Nat) {
        j >= Nat.1 and p.coeff(j) != Complex.0
    }
} by {
    if exists(x: Complex, y: Complex) { polynomial_eval(p, x) != polynomial_eval(p, y) } {
        if not exists(j: Nat) { j >= Nat.1 and p.coeff(j) != Complex.0 } {
            forall(j: Nat) {
                if j >= Nat.1 {
                    if p.coeff(j) != Complex.0 {
                        j >= Nat.1 and p.coeff(j) != Complex.0
                        exists(t: Nat) { t >= Nat.1 and p.coeff(t) != Complex.0 }
                        false
                    }
                    p.coeff(j) = Complex.0
                }
            }
            polynomial_support_bound_exists(p)
            let n: Nat satisfy {
                polynomial_support_bounded_by(p, n)
            }
            forall(w: Complex) {
                polynomial_eval_eq_range_sum(p, w, n)
                polynomial_eval(p, w) = range_sum(coeff_pow_term(p.coeff, w), n)
                if n >= Nat.1 {
                    forall(i: Nat) {
                        if i >= Nat.1 and i < n {
                            p.coeff(i) = Complex.0
                            coeff_pow_term(p.coeff, w, i) = p.coeff(i) * w.pow(i)
                            coeff_pow_term(p.coeff, w, i) = Complex.0 * w.pow(i)
                            mul_zero_left(w.pow(i))
                            Complex.0 * w.pow(i) = Complex.0
                            coeff_pow_term(p.coeff, w, i) = Complex.0
                        }
                    }
                    range_sum_zero_above_zero(coeff_pow_term(p.coeff, w), n)
                    n >= Nat.1 and (forall(i: Nat) { i >= Nat.1 and i < n implies coeff_pow_term(p.coeff, w, i) = Complex.0 })
                    range_sum(coeff_pow_term(p.coeff, w), n) = coeff_pow_term(p.coeff, w, Nat.0)
                    coeff_pow_term(p.coeff, w, Nat.0) = p.coeff(Nat.0) * w.pow(Nat.0)
                    pow_zero(w)
                    w.pow(Nat.0) = Complex.1
                    p.coeff(Nat.0) * Complex.1 = p.coeff(Nat.0)
                    coeff_pow_term(p.coeff, w, Nat.0) = p.coeff(Nat.0)
                    range_sum(coeff_pow_term(p.coeff, w), n) = p.coeff(Nat.0)
                    polynomial_eval(p, w) = p.coeff(Nat.0)
                } else {
                    not n >= Nat.1
                    nat_not_ge_one_imp_zero(n)
                    n = Nat.0
                    range_sum_zero(coeff_pow_term(p.coeff, w))
                    range_sum(coeff_pow_term(p.coeff, w), Nat.0) = Complex.0
                    polynomial_eval(p, w) = range_sum(coeff_pow_term(p.coeff, w), n)
                    polynomial_eval(p, w) = Complex.0
                    polynomial_support_bounded_by_apply(p, Nat.0, Nat.0)
                    not Nat.0 < Nat.0 implies p.coeff(Nat.0) = Complex.0
                    lt_not_symm(Nat.0, Nat.0)
                    not Nat.0 < Nat.0
                    p.coeff(Nat.0) = Complex.0
                    polynomial_eval(p, w) = p.coeff(Nat.0)
                }
            }
            exists(x: Complex, y: Complex) { polynomial_eval(p, x) != polynomial_eval(p, y) }
            let x0: Complex satisfy {
                exists(y0: Complex) { polynomial_eval(p, x0) != polynomial_eval(p, y0) }
            }
            let y0: Complex satisfy {
                polynomial_eval(p, x0) != polynomial_eval(p, y0)
            }
            polynomial_eval(p, x0) = p.coeff(Nat.0)
            polynomial_eval(p, y0) = p.coeff(Nat.0)
            polynomial_eval(p, x0) = polynomial_eval(p, y0)
            false
        }
        exists(j: Nat) { j >= Nat.1 and p.coeff(j) != Complex.0 }
    }
}

/// The least support bound of a polynomial: the least `N` with
/// `polynomial_support_bounded_by(p, N)`.
let polynomial_least_bound(p: Polynomial[Complex]) -> n0: Nat satisfy {
    polynomial_support_bounded_by(p, n0) and
    (forall(m: Nat) { polynomial_support_bounded_by(p, m) implies n0 <= m })
} by {
    polynomial_support_bound_exists(p)
    let n: Nat satisfy {
        polynomial_support_bounded_by(p, n)
    }
    exists_least_nat_witness(polynomial_support_bounded_by(p))
    let m: Nat satisfy {
        is_least_nat_witness(polynomial_support_bounded_by(p), m)
    }
    let candidate = m
    polynomial_support_bounded_by(p, m)
    forall(k: Nat) {
        if polynomial_support_bounded_by(p, k) {
            is_least_nat_witness(polynomial_support_bounded_by(p), m) = (polynomial_support_bounded_by(p, m) and forall(j: Nat) { polynomial_support_bounded_by(p, j) implies m <= j })
            forall(j: Nat) { polynomial_support_bounded_by(p, j) implies m <= j }
            polynomial_support_bounded_by(p, k) implies m <= k
            m <= k
        }
    }
    candidate = m and polynomial_support_bounded_by(p, candidate) and forall(k: Nat) {
        polynomial_support_bounded_by(p, k) implies candidate <= k
    }
}

/// The least support bound bounds the support.
theorem polynomial_least_bound_props(p: Polynomial[Complex]) {
    polynomial_support_bounded_by(p, polynomial_least_bound(p)) and
    (forall(m: Nat) { polynomial_support_bounded_by(p, m) implies polynomial_least_bound(p) <= m })
}

/// A support bound at least one with a vanishing coefficient just below it
/// descends to the predecessor.
theorem polynomial_support_bound_pred_of_coeff_zero(p: Polynomial[Complex], nb: Nat) {
    polynomial_support_bounded_by(p, nb) and nb >= Nat.1 and p.coeff(nb - 1) = Complex.0 implies polynomial_support_bounded_by(p, nb - 1)
} by {
    if polynomial_support_bounded_by(p, nb) and nb >= Nat.1 and p.coeff(nb - 1) = Complex.0 {
        polynomial_support_bounded_by(p, nb) = coeff_zero_from(p.coeff, nb)
        coeff_zero_from(p.coeff, nb)
        coeff_zero_from(p.coeff, nb) = forall(k: Nat) {
            coeff_zero_from_at(p.coeff, nb, k)
        }
        forall(k: Nat) {
            coeff_zero_from_at(p.coeff, nb, k)
        }
        forall(k: Nat) {
            if not k < nb - 1 {
                lt_or_lte(k, nb - 1)
                if k < nb - 1 {
                    false
                }
                nb - 1 <= k
                if k < nb {
                    nat_lte_pred_of_lt(k, nb)
                    k <= nb - 1
                    lte_antisymm(k, nb - 1)
                    k = nb - 1
                    p.coeff(k) = p.coeff(nb - 1)
                    p.coeff(k) = Complex.0
                    coeff_zero_at(p.coeff, k)
                    coeff_zero_from_at(p.coeff, nb - 1, k) = (not k < nb - 1 implies coeff_zero_at(p.coeff, k))
                    coeff_zero_from_at(p.coeff, nb - 1, k)
                } else {
                    not k < nb
                    coeff_zero_from_at(p.coeff, nb, k) = (not k < nb implies coeff_zero_at(p.coeff, k))
                    coeff_zero_at(p.coeff, k)
                    coeff_zero_from_at(p.coeff, nb - 1, k) = (not k < nb - 1 implies coeff_zero_at(p.coeff, k))
                    coeff_zero_from_at(p.coeff, nb - 1, k)
                }
            }
        }
        coeff_zero_from(p.coeff, nb - 1) = forall(k: Nat) {
            coeff_zero_from_at(p.coeff, nb - 1, k)
        }
        coeff_zero_from(p.coeff, nb - 1)
        polynomial_support_bounded_by(p, nb - 1) = coeff_zero_from(p.coeff, nb - 1)
        polynomial_support_bounded_by(p, nb - 1)
    }
}

/// The coefficient just below the least support bound is nonzero, when that
/// bound is positive.
theorem polynomial_leading_coeff_nonzero(p: Polynomial[Complex]) {
    polynomial_least_bound(p) >= Nat.1 implies p.coeff(polynomial_least_bound(p) - 1) != Complex.0
} by {
    if polynomial_least_bound(p) >= Nat.1 {
        if p.coeff(polynomial_least_bound(p) - 1) = Complex.0 {
            polynomial_least_bound_props(p)
            polynomial_support_bounded_by(p, polynomial_least_bound(p))
            polynomial_support_bound_pred_of_coeff_zero(p, polynomial_least_bound(p))
            polynomial_support_bounded_by(p, polynomial_least_bound(p) - 1)
            polynomial_least_bound_props(p)
            forall(m: Nat) { polynomial_support_bounded_by(p, m) implies polynomial_least_bound(p) <= m }
            polynomial_support_bounded_by(p, polynomial_least_bound(p) - 1) implies polynomial_least_bound(p) <= polynomial_least_bound(p) - 1
            polynomial_least_bound(p) <= polynomial_least_bound(p) - 1
            polynomial_least_bound(p) - 1 < polynomial_least_bound(p)
            lt_suc(polynomial_least_bound(p) - 1)
            (polynomial_least_bound(p) - 1).suc = polynomial_least_bound(p)
            polynomial_least_bound(p) - 1 < polynomial_least_bound(p)
            false
        }
        p.coeff(polynomial_least_bound(p) - 1) != Complex.0
    }
}

/// A nonconstant polynomial has a positive index whose coefficient is
/// nonzero and above which all coefficients vanish.
theorem nonconstant_has_leading_coefficient(p: Polynomial[Complex]) {
    exists(x: Complex, y: Complex) { polynomial_eval(p, x) != polynomial_eval(p, y) } implies exists(n: Nat) {
        n >= Nat.1 and p.coeff(n) != Complex.0 and polynomial_support_bounded_by(p, n.suc)
    }
} by {
    if exists(x: Complex, y: Complex) { polynomial_eval(p, x) != polynomial_eval(p, y) } {
        let nb: Nat = polynomial_least_bound(p)
        nonconstant_has_nonzero_high_coeff(p)
        exists(j: Nat) { j >= Nat.1 and p.coeff(j) != Complex.0 }
        let j0: Nat satisfy {
            j0 >= Nat.1 and p.coeff(j0) != Complex.0
        }
        polynomial_least_bound_props(p)
        polynomial_support_bounded_by(p, nb)
        forall(m: Nat) { polynomial_support_bounded_by(p, m) implies nb <= m }
        if nb <= j0 {
            polynomial_support_bounded_by(p, nb) implies nb <= j0
            polynomial_support_bounded_by(p, nb)
            nb <= j0
            if p.coeff(j0) = Complex.0 {
                false
            }
            p.coeff(j0) != Complex.0
            polynomial_support_bounded_by_apply(p, nb, j0)
            nb <= j0
            not j0 < nb
            polynomial_support_bounded_by_apply(p, nb, j0) = (not j0 < nb implies p.coeff(j0) = Complex.0)
            p.coeff(j0) = Complex.0
            false
        }
        not nb <= j0
        j0 < nb
        j0 >= Nat.1
        lt_suc(j0)
        j0 < j0.suc
        lt_imp_lte_suc(j0, nb)
        j0.suc <= nb
        lte_trans(j0, j0.suc, nb)
        j0 <= nb
        Nat.1 <= j0
        lte_trans(Nat.1, j0, nb)
        nb >= Nat.1
        polynomial_leading_coeff_nonzero(p)
        p.coeff(nb - 1) != Complex.0
        nat_lte_pred_of_lt(j0, nb)
        j0 <= nb - 1
        Nat.1 <= j0
        lte_trans(Nat.1, j0, nb - 1)
        nb - 1 >= Nat.1
        let n: Nat = nb - 1
        nb = polynomial_least_bound(p)
        polynomial_support_bounded_by(p, nb)
        nat_suc_pred_self(nb)
        (nb - 1).suc = nb
        n = nb - 1
        n.suc = nb
        polynomial_support_bounded_by(p, n.suc)
        n >= Nat.1
        n = nb - 1
        p.coeff(n) != Complex.0
        n >= Nat.1 and p.coeff(n) != Complex.0 and polynomial_support_bounded_by(p, n.suc)
        exists(t: Nat) {
            t >= Nat.1 and p.coeff(t) != Complex.0 and polynomial_support_bounded_by(p, t.suc)
        }
    }
}

// ---------------------------------------------------------------------------
// Section 5.  Growth of |p(z)| at infinity.
// ---------------------------------------------------------------------------

/// A real number strictly greater than zero has a nonzero inverse, and the
/// inverse is again positive.
theorem real_inv_pos(a: Real) {
    a > Real.0 implies a.inverse > Real.0
} by {
    if a > Real.0 {
        a.is_positive
        if not a.inverse.is_positive {
            a.inverse <= Real.0
            mul_le_mul_of_nonneg_left(a.inverse, Real.0, a)
            a.inverse <= Real.0 and a >= Real.0 implies a * a.inverse <= a * Real.0
            a >= Real.0
            a * a.inverse <= a * Real.0
            a * Real.0 = Real.0
            a * a.inverse <= Real.0
            a * a.inverse = Real.1
            Real.1 <= Real.0
            real_one_positive
            Real.1 > Real.0
            false
        }
        a.inverse.is_positive
        a.inverse.is_positive implies a.inverse > Real.0
        a.inverse > Real.0
    }
}

/// A positive real multiplied by its inverse is one.
theorem real_mul_inv_self(a: Real) {
    a != Real.0 implies a * a.inverse = Real.1
} by {
    if a != Real.0 {
        a * a.inverse = Real.1
    }
}

/// The modulus of the polynomial evaluation is at least `M` outside a large
/// disk, when `n >= 1` is an index with a nonzero coefficient `p.coeff(n)`
/// and all coefficients above `n` vanish.
theorem polynomial_eval_growth(p: Polynomial[Complex], n: Nat, m0: Real) {
    n >= Nat.1 and p.coeff(n) != Complex.0 and polynomial_support_bounded_by(p, n.suc) and m0 >= Real.0
    implies exists(r: Real) {
        r >= Real.0 and forall(z: Complex) {
            z.modulus >= r implies polynomial_eval(p, z).modulus >= m0
        }
    }
} by {
    if n >= Nat.1 and p.coeff(n) != Complex.0 and polynomial_support_bounded_by(p, n.suc) and m0 >= Real.0 {
        let a: Real = p.coeff(n).modulus
        let s: Real = coefficient_modulus_sum(p, n)
        modulus_pos(p.coeff(n))
        p.coeff(n) != Complex.0 implies p.coeff(n).modulus > Real.0
        a > Real.0
        real_inv_pos(a)
        a.inverse > Real.0
        a.inverse >= Real.0
        forall(i: Nat) {
            if i < n {
                one_pow[Real](i)
                Real.1.pow(i) = Real.1
                coeff_pow_bound_term(p.coeff, Real.1, i) = p.coeff(i).modulus * Real.1.pow(i)
                p.coeff(i).modulus * Real.1 = p.coeff(i).modulus
                coeff_pow_bound_term(p.coeff, Real.1, i) = p.coeff(i).modulus
                modulus_nonneg(p.coeff(i))
                p.coeff(i).modulus >= Real.0
            }
        }
        range_sum_nonneg_real(coeff_pow_bound_term(p.coeff, Real.1), n)
        forall(i: Nat) {
            if i < n {
                one_pow[Real](i)
                Real.1.pow(i) = Real.1
                coeff_pow_bound_term(p.coeff, Real.1, i) = p.coeff(i).modulus * Real.1.pow(i)
                p.coeff(i).modulus * Real.1 = p.coeff(i).modulus
                coeff_pow_bound_term(p.coeff, Real.1, i) = p.coeff(i).modulus
                coeff_pow_bound_term(p.coeff, Real.1, i) >= Real.0
            }
        }
        s = coefficient_modulus_sum(p, n)
        coefficient_modulus_sum(p, n) = range_sum(coeff_pow_bound_term(p.coeff, Real.1), n)
        s = range_sum(coeff_pow_bound_term(p.coeff, Real.1), n)
        forall(i: Nat) { i < n implies coeff_pow_bound_term(p.coeff, Real.1, i) >= Real.0 }
        range_sum(coeff_pow_bound_term(p.coeff, Real.1), n) >= Real.0
        s >= Real.0
        m0 >= Real.0
        add_lte_add(Real.0, m0, Real.0, s)
        Real.0 <= m0 and Real.0 <= s implies Real.0 + Real.0 <= m0 + s
        Real.0 <= m0
        Real.0 <= s
        Real.0 + Real.0 <= m0 + s
        Real.0 + Real.0 = Real.0
        Real.0 <= m0 + s
        m0 + s >= Real.0
        mul_nonneg(m0 + s, a.inverse)
        m0 + s >= Real.0 and a.inverse >= Real.0 implies (m0 + s) * a.inverse >= Real.0
        (m0 + s) * a.inverse >= Real.0
        let r: Real = (m0 + s) * a.inverse + Real.1
        lte_add_right(Real.0, (m0 + s) * a.inverse, Real.1)
        Real.0 <= (m0 + s) * a.inverse implies Real.0 + Real.1 <= (m0 + s) * a.inverse + Real.1
        Real.0 + Real.1 <= r
        Real.0 + Real.1 = Real.1
        Real.1 <= r
        r >= Real.1
        real_one_positive
        Real.1 > Real.0
        lt_imp_lte(Real.0, Real.1)
        Real.0 <= Real.1
        lte_trans(Real.0, Real.1, r)
        Real.0 <= r
        forall(z: Complex) {
            if z.modulus >= r {
                r <= z.modulus
                r >= Real.1
                lte_trans(Real.1, r, z.modulus)
                z.modulus >= Real.1
                r = (m0 + s) * a.inverse + Real.1
                lte_self((m0 + s) * a.inverse)
                (m0 + s) * a.inverse <= (m0 + s) * a.inverse
                Real.0 <= Real.1
                add_lte_add((m0 + s) * a.inverse, (m0 + s) * a.inverse, Real.0, Real.1)
                (m0 + s) * a.inverse <= (m0 + s) * a.inverse and Real.0 <= Real.1 implies (m0 + s) * a.inverse + Real.0 <= (m0 + s) * a.inverse + Real.1
                (m0 + s) * a.inverse + Real.0 <= (m0 + s) * a.inverse + Real.1
                (m0 + s) * a.inverse + Real.0 = (m0 + s) * a.inverse
                (m0 + s) * a.inverse <= (m0 + s) * a.inverse + Real.1
                (m0 + s) * a.inverse <= r
                lte_trans((m0 + s) * a.inverse, r, z.modulus)
                (m0 + s) * a.inverse <= z.modulus
                a != Real.0
                a * a.inverse = Real.1
                a * ((m0 + s) * a.inverse) = (m0 + s) * (a * a.inverse)
                (m0 + s) * (a * a.inverse) = (m0 + s) * Real.1
                (m0 + s) * Real.1 = m0 + s
                a * ((m0 + s) * a.inverse) = m0 + s
                mul_le_mul_of_nonneg_left((m0 + s) * a.inverse, z.modulus, a)
                (m0 + s) * a.inverse <= z.modulus and a >= Real.0 implies a * ((m0 + s) * a.inverse) <= a * z.modulus
                a >= Real.0
                a * ((m0 + s) * a.inverse) <= a * z.modulus
                m0 + s <= a * z.modulus
                polynomial_eval_eq_range_sum(p, z, n.suc)
                polynomial_eval(p, z) = range_sum(coeff_pow_term(p.coeff, z), n.suc)
                range_sum_suc(coeff_pow_term(p.coeff, z), n)
                range_sum(coeff_pow_term(p.coeff, z), n.suc) = range_sum(coeff_pow_term(p.coeff, z), n) + coeff_pow_term(p.coeff, z, n)
                coeff_pow_term(p.coeff, z, n) = p.coeff(n) * z.pow(n)
                polynomial_eval(p, z) = range_sum(coeff_pow_term(p.coeff, z), n) + p.coeff(n) * z.pow(n)
                modulus_add_lower(p.coeff(n) * z.pow(n), range_sum(coeff_pow_term(p.coeff, z), n))
                (p.coeff(n) * z.pow(n) + range_sum(coeff_pow_term(p.coeff, z), n)).modulus >= (p.coeff(n) * z.pow(n)).modulus - (range_sum(coeff_pow_term(p.coeff, z), n)).modulus
                polynomial_eval(p, z).modulus >= (p.coeff(n) * z.pow(n)).modulus - (range_sum(coeff_pow_term(p.coeff, z), n)).modulus
                coeff_pow_term_modulus(p.coeff(n), z, n)
                (p.coeff(n) * z.pow(n)).modulus = a * z.modulus.pow(n)
                polynomial_eval(p, z).modulus >= a * z.modulus.pow(n) - (range_sum(coeff_pow_term(p.coeff, z), n)).modulus
                polynomial_eval_tail_bound(p, z, n)
                n >= Nat.1 and z.modulus >= Real.1
                n >= Nat.1
                z.modulus >= Real.1
                (range_sum(coeff_pow_term(p.coeff, z), n)).modulus <= z.modulus.pow(n - 1) * coefficient_modulus_sum(p, n)
                coefficient_modulus_sum(p, n) = range_sum(coeff_pow_bound_term(p.coeff, Real.1), n)
                s = range_sum(coeff_pow_bound_term(p.coeff, Real.1), n)
                (range_sum(coeff_pow_term(p.coeff, z), n)).modulus <= z.modulus.pow(n - 1) * s
                real_sub_lte_lte(a * z.modulus.pow(n), (range_sum(coeff_pow_term(p.coeff, z), n)).modulus, z.modulus.pow(n - 1) * s)
                (range_sum(coeff_pow_term(p.coeff, z), n)).modulus <= z.modulus.pow(n - 1) * s implies a * z.modulus.pow(n) - z.modulus.pow(n - 1) * s <= a * z.modulus.pow(n) - (range_sum(coeff_pow_term(p.coeff, z), n)).modulus
                a * z.modulus.pow(n) - z.modulus.pow(n - 1) * s <= a * z.modulus.pow(n) - (range_sum(coeff_pow_term(p.coeff, z), n)).modulus
                a * z.modulus.pow(n) - (range_sum(coeff_pow_term(p.coeff, z), n)).modulus <= polynomial_eval(p, z).modulus
                lte_trans(a * z.modulus.pow(n) - z.modulus.pow(n - 1) * s, a * z.modulus.pow(n) - (range_sum(coeff_pow_term(p.coeff, z), n)).modulus, polynomial_eval(p, z).modulus)
                polynomial_eval(p, z).modulus >= a * z.modulus.pow(n) - z.modulus.pow(n - 1) * s
                nat_suc_pred_self(n)
                (n - 1).suc = n
                real_pow_suc(z.modulus, n - 1)
                z.modulus.pow(n) = z.modulus.pow(n - 1) * z.modulus
                a * z.modulus.pow(n) = a * (z.modulus.pow(n - 1) * z.modulus)
                a * z.modulus.pow(n) - z.modulus.pow(n - 1) * s = z.modulus.pow(n - 1) * (a * z.modulus - s)
                polynomial_eval(p, z).modulus >= z.modulus.pow(n - 1) * (a * z.modulus - s)
                lte_add_right(m0 + s, a * z.modulus, -s)
                m0 + s <= a * z.modulus implies m0 + s + -s <= a * z.modulus + -s
                m0 + s + -s = m0 + s - s
                sub_cancels(m0, s)
                m0 + s - s = m0
                m0 + s + -s = m0
                m0 <= a * z.modulus + -s
                a * z.modulus + -s = a * z.modulus - s
                m0 <= a * z.modulus - s
                a * z.modulus - s >= m0
                real_pow_ge_one(z.modulus, n - 1)
                z.modulus >= Real.1
                z.modulus.pow(n - 1) >= Real.1
                mul_le_mul_of_nonneg_right(Real.1, z.modulus.pow(n - 1), m0)
                Real.1 <= z.modulus.pow(n - 1) and m0 >= Real.0 implies Real.1 * m0 <= z.modulus.pow(n - 1) * m0
                m0 >= Real.0
                Real.1 * m0 <= z.modulus.pow(n - 1) * m0
                Real.1 * m0 = m0
                m0 <= z.modulus.pow(n - 1) * m0
                mul_le_mul_of_nonneg_left(m0, a * z.modulus - s, z.modulus.pow(n - 1))
                m0 <= a * z.modulus - s and z.modulus.pow(n - 1) >= Real.0 implies z.modulus.pow(n - 1) * m0 <= z.modulus.pow(n - 1) * (a * z.modulus - s)
                z.modulus.pow(n - 1) >= Real.0
                z.modulus.pow(n - 1) * m0 <= z.modulus.pow(n - 1) * (a * z.modulus - s)
                lte_trans(m0, z.modulus.pow(n - 1) * m0, z.modulus.pow(n - 1) * (a * z.modulus - s))
                m0 <= z.modulus.pow(n - 1) * (a * z.modulus - s)
                z.modulus.pow(n - 1) * (a * z.modulus - s) <= polynomial_eval(p, z).modulus
                lte_trans(m0, z.modulus.pow(n - 1) * (a * z.modulus - s), polynomial_eval(p, z).modulus)
                polynomial_eval(p, z).modulus >= m0
            }
        }
        r >= Real.0
        forall(z: Complex) {
            z.modulus >= r implies polynomial_eval(p, z).modulus >= m0
        }
        r >= Real.0 and forall(z: Complex) {
            z.modulus >= r implies polynomial_eval(p, z).modulus >= m0
        }
        exists(rt: Real) {
            rt >= Real.0 and forall(z: Complex) {
                z.modulus >= rt implies polynomial_eval(p, z).modulus >= m0
            }
        }
    }
}

// ---------------------------------------------------------------------------
// Section 6.  The perturbation lemma (D'Alembert's lemma).
// strictly smaller modulus.  The proof expands `p(z0 + w)` around `z0` by
// iterated synthetic division by `X - z0` (the remainder theorem), isolates
// the first nonzero shifted coefficient, and perturbs along a direction
// `u` chosen among the powers of `omega(4k)`, whose `k`-th powers are the
// fourth roots of unity; one of the four directions decreases the modulus.
// ---------------------------------------------------------------------------

/// A natural strictly below `n` has `n - j` at least one.
theorem nat_sub_ge_one(j: Nat, n: Nat) {
    j < n implies n - j >= Nat.1
} by {
    if j < n {
        lt_imp_lte_suc(j, n)
        j.suc <= n
        lt_suc(j)
        j < j.suc
        j <= j.suc
        lte_trans(j, j.suc, n)
        j <= n
        add_sub(j, n)
        n - j + j = n
        add_comm(n - j, j)
        j + (n - j) = n
        if n - j = Nat.0 {
            j + Nat.0 = n
            j + Nat.0 = j
            j = n
            lt_not_symm(j, j)
            not j < j
            j < j
            false
        }
        n - j != Nat.0
        zero_or_suc(n - j)
        let m: Nat satisfy {
            n - j = m.suc
        }
        n - j = m.suc
        m.suc >= Nat.1
        n - j >= Nat.1
    }
}

/// Subtracting one more lowers a bounded subtraction by one:
/// `n - j - 1 = n - j.suc` for `j < n`.
theorem nat_sub_sub_suc(j: Nat, n: Nat) {
    j < n implies n - j - 1 = n - j.suc
} by {
    if j < n {
        lt_imp_lte_suc(j, n)
        j.suc <= n
        add_comm(j.suc, n - j.suc)
        j.suc + (n - j.suc) = n - j.suc + j.suc
        add_sub(j.suc, n)
        n - j.suc + j.suc = n
        j.suc + (n - j.suc) = n
        add_imp_sub_left(j, n - j.suc + 1, n)
        j + (n - j.suc + 1) = n
        add_imp_sub_left(j, n - j.suc + 1, n)
        n - j = n - j.suc + 1
        n - j - 1 = n - j.suc + 1 - 1
        n - j.suc + 1 - 1 = n - j.suc
        n - j - 1 = n - j.suc
    }
}

/// The `j`-fold iterated synthetic quotient of `p` by `X - a`, dividing at
/// the fixed support bound `n` at every step.
define iterated_quotient[F: Field](p: Polynomial[F], a: F, n: Nat, j: Nat) -> Polynomial[F] {
    match j {
        Nat.zero {
            p
        }
        Nat.suc(pred) {
            polynomial_quotient(iterated_quotient(p, a, n, pred), a, n - pred)
        }
    }
}

/// The zeroth iterated quotient is the polynomial itself.
theorem iterated_quotient_zero[F: Field](p: Polynomial[F], a: F, n: Nat) {
    iterated_quotient(p, a, n, Nat.0) = p
} by {
    iterated_quotient(p, a, n, Nat.0) = p
}

/// The successor iterated quotient divides the previous one.
theorem iterated_quotient_suc[F: Field](p: Polynomial[F], a: F, n: Nat, j: Nat) {
    iterated_quotient(p, a, n, j.suc) =
        polynomial_quotient(iterated_quotient(p, a, n, j), a, n - j)
} by {
    iterated_quotient(p, a, n, j.suc) =
        polynomial_quotient(iterated_quotient(p, a, n, j), a, n - j)
}

/// The `j`-th iterated quotient is supported below `n - j`.
theorem iterated_quotient_support_bounded[F: Field](p: Polynomial[F], a: F, n: Nat, j: Nat) {
    polynomial_support_bounded_by(p, n) and j <= n
    implies polynomial_support_bounded_by(iterated_quotient(p, a, n, j), n - j)
} by {
    define st(k: Nat) -> Bool {
        polynomial_support_bounded_by(p, n) and k <= n
        implies polynomial_support_bounded_by(iterated_quotient(p, a, n, k), n - k)
    }
    iterated_quotient_zero(p, a, n)
    iterated_quotient(p, a, n, Nat.0) = p
    forall(t: Nat) {
        if polynomial_support_bounded_by(p, n) and Nat.0 <= n {
            polynomial_support_bounded_by(p, n - Nat.0)
            polynomial_support_bounded_by(iterated_quotient(p, a, n, Nat.0), n - Nat.0)
        }
    }
    st(Nat.0)
    forall(k: Nat) {
        if st(k) {
            if polynomial_support_bounded_by(p, n) and k.suc <= n {
                lt_suc(k)
                k < k.suc
                k <= k.suc
                k.suc <= n
                lte_trans(k, k.suc, n)
                k <= n
                lt_lte_trans(k, k.suc, n)
                k < n
                st(k) = (polynomial_support_bounded_by(p, n) and k <= n
                    implies polynomial_support_bounded_by(iterated_quotient(p, a, n, k), n - k))
                polynomial_support_bounded_by(iterated_quotient(p, a, n, k), n - k)
                nat_sub_ge_one(k, n)
                n - k >= Nat.1
                iterated_quotient_suc(p, a, n, k)
                iterated_quotient(p, a, n, k.suc) = polynomial_quotient(iterated_quotient(p, a, n, k), a, n - k)
                polynomial_quotient_support_bounded_by_pred(iterated_quotient(p, a, n, k), a, n - k - 1)
                nat_suc_pred_self(n - k)
                (n - k - 1).suc = n - k
                polynomial_support_bounded_by(polynomial_quotient(iterated_quotient(p, a, n, k), a, n - k), n - k - 1)
                polynomial_support_bounded_by(iterated_quotient(p, a, n, k.suc), n - k - 1)
                nat_sub_sub_suc(k, n)
                n - k - 1 = n - k.suc
                polynomial_support_bounded_by(iterated_quotient(p, a, n, k.suc), n - k.suc)
            }
            st(k.suc)
        }
    }
    st(Nat.0) and forall(k: Nat) {
        st(k) implies st(k.suc)
    }
    alt_induction(st)
    st(j)
    polynomial_support_bounded_by(p, n) and j <= n
    polynomial_support_bounded_by(iterated_quotient(p, a, n, j), n - j)
}

/// The remainder theorem at a bound `m`, in evaluation form:
/// `p(x) = p(a) + (x - a) * q_1(x)`.
theorem polynomial_remainder_global(p: Polynomial[Complex], a: Complex, x: Complex, m: Nat) {
    polynomial_support_bounded_by(p, m)
    implies polynomial_eval(p, x) = polynomial_eval(p, a) + (x - a) * polynomial_eval(iterated_quotient(p, a, m, Nat.1), x)
} by {
    if polynomial_support_bounded_by(p, m) {
        polynomial_remainder_theorem_bundled_bound(p, a, x, m)
        polynomial_eval_bound(p, x, m) = polynomial_eval_bound(p, a, m) + (x - a) * polynomial_eval_bound(polynomial_quotient(p, a, m), x, m)
        polynomial_eval_eq_eval_bound_of_support_bounded(p, x, m)
        polynomial_eval(p, x) = polynomial_eval_bound(p, x, m)
        polynomial_eval_eq_eval_bound_of_support_bounded(p, a, m)
        polynomial_eval(p, a) = polynomial_eval_bound(p, a, m)
        polynomial_quotient_support_bounded_by(p, a, m)
        polynomial_support_bounded_by(polynomial_quotient(p, a, m), m)
        polynomial_eval_eq_eval_bound_of_support_bounded(polynomial_quotient(p, a, m), x, m)
        polynomial_eval(polynomial_quotient(p, a, m), x) = polynomial_eval_bound(polynomial_quotient(p, a, m), x, m)
        iterated_quotient_suc(p, a, m, Nat.0)
        iterated_quotient(p, a, m, Nat.1) = polynomial_quotient(iterated_quotient(p, a, m, Nat.0), a, m)
        iterated_quotient_zero(p, a, m)
        iterated_quotient(p, a, m, Nat.0) = p
        iterated_quotient(p, a, m, Nat.1) = polynomial_quotient(p, a, m)
        polynomial_eval(p, x) = polynomial_eval(p, a) + (x - a) * polynomial_eval(iterated_quotient(p, a, m, Nat.1), x)
    }
}

/// The iterated-quotient expansion of `p(x)` around `a`:
/// `p(x) = p(a) + sum_{j < k} q_{j+1}(a) * (x - a)^(j+1) + (x - a)^(k+1) * q_{k+1}(x)`.
theorem polynomial_eval_iterated_expansion(p: Polynomial[Complex], a: Complex, x: Complex, n: Nat, k: Nat) {
    polynomial_support_bounded_by(p, n) and k <= n
    implies polynomial_eval(p, x) = polynomial_eval(p, a) +
        range_sum(function(j: Nat) { polynomial_eval(iterated_quotient(p, a, n, j.suc), a) * (x - a).pow(j.suc) }, k) + (x - a).pow(k.suc) * polynomial_eval(iterated_quotient(p, a, n, k.suc), x)
} by {
    define st(m: Nat) -> Bool {
        polynomial_support_bounded_by(p, n) and m <= n
        implies polynomial_eval(p, x) = polynomial_eval(p, a) +
            range_sum(function(j: Nat) { polynomial_eval(iterated_quotient(p, a, n, j.suc), a) * (x - a).pow(j.suc) }, m) + (x - a).pow(m.suc) * polynomial_eval(iterated_quotient(p, a, n, m.suc), x)
    }
    range_sum_zero(function(j: Nat) { polynomial_eval(iterated_quotient(p, a, n, j.suc), a) * (x - a).pow(j.suc) })
    forall(t: Nat) {
        if polynomial_support_bounded_by(p, n) and Nat.0 <= n {
            polynomial_remainder_global(p, a, x, n)
            polynomial_eval(p, x) = polynomial_eval(p, a) + (x - a) * polynomial_eval(iterated_quotient(p, a, n, Nat.1), x)
            range_sum(function(j: Nat) { polynomial_eval(iterated_quotient(p, a, n, j.suc), a) * (x - a).pow(j.suc) }, Nat.0) = Complex.0
            polynomial_eval(p, a) + Complex.0 = polynomial_eval(p, a)
            complex_pow_suc(x - a, Nat.0)
            (x - a).pow(Nat.0.suc) = (x - a) * (x - a).pow(Nat.0)
            pow_zero(x - a)
            (x - a).pow(Nat.0) = Complex.1
            mul_one_right(x - a)
            (x - a) * Complex.1 = x - a
            (x - a).pow(Nat.0.suc) = x - a
            polynomial_eval(p, x) = polynomial_eval(p, a) + range_sum(function(j: Nat) { polynomial_eval(iterated_quotient(p, a, n, j.suc), a) * (x - a).pow(j.suc) }, Nat.0) + (x - a).pow(Nat.0.suc) * polynomial_eval(iterated_quotient(p, a, n, Nat.0.suc), x)
            st(Nat.0)
        }
    }
    forall(m: Nat) {
        if st(m) {
            if polynomial_support_bounded_by(p, n) and m.suc <= n {
                lte_trans(m, m.suc, n)
                lt_suc(m)
                m < m.suc
                lt_lte_trans(m, m.suc, n)
                m < n
                m <= n
                st(m) = (polynomial_support_bounded_by(p, n) and m <= n
                    implies polynomial_eval(p, x) = polynomial_eval(p, a) +
                        range_sum(function(j: Nat) { polynomial_eval(iterated_quotient(p, a, n, j.suc), a) * (x - a).pow(j.suc) }, m) + (x - a).pow(m.suc) * polynomial_eval(iterated_quotient(p, a, n, m.suc), x))
                polynomial_eval(p, x) = polynomial_eval(p, a) +
                    range_sum(function(j: Nat) { polynomial_eval(iterated_quotient(p, a, n, j.suc), a) * (x - a).pow(j.suc) }, m) + (x - a).pow(m.suc) * polynomial_eval(iterated_quotient(p, a, n, m.suc), x)
                let qs: Polynomial[Complex] = iterated_quotient(p, a, n, m.suc)
                m.suc <= n
                iterated_quotient_support_bounded(p, a, n, m.suc)
                polynomial_support_bounded_by(qs, n - m.suc)
                polynomial_remainder_global(qs, a, x, n - m.suc)
                polynomial_eval(qs, x) = polynomial_eval(qs, a) + (x - a) * polynomial_eval(iterated_quotient(qs, a, n - m.suc, Nat.1), x)
                iterated_quotient_zero(qs, a, n - m.suc)
                iterated_quotient(qs, a, n - m.suc, Nat.0) = qs
                iterated_quotient_suc(qs, a, n - m.suc, Nat.0)
                iterated_quotient(qs, a, n - m.suc, Nat.1) = polynomial_quotient(iterated_quotient(qs, a, n - m.suc, Nat.0), a, n - m.suc)
                iterated_quotient(qs, a, n - m.suc, Nat.1) = polynomial_quotient(qs, a, n - m.suc)
                qs = iterated_quotient(p, a, n, m.suc)
                iterated_quotient(qs, a, n - m.suc, Nat.1) = polynomial_quotient(iterated_quotient(p, a, n, m.suc), a, n - m.suc)
                iterated_quotient_suc(p, a, n, m.suc)
                iterated_quotient(p, a, n, m.suc.suc) = polynomial_quotient(iterated_quotient(p, a, n, m.suc), a, n - m.suc)
                iterated_quotient(p, a, n, m.suc.suc) = iterated_quotient(qs, a, n - m.suc, Nat.1)
                polynomial_eval(qs, x) = polynomial_eval(qs, a) + (x - a) * polynomial_eval(iterated_quotient(p, a, n, m.suc.suc), x)
                complex_pow_suc(x - a, m.suc)
                (x - a).pow(m.suc.suc) = (x - a) * (x - a).pow(m.suc)
                mul_comm((x - a).pow(m.suc), x - a)
                (x - a).pow(m.suc) * (x - a) = (x - a) * (x - a).pow(m.suc)
                (x - a).pow(m.suc.suc) = (x - a).pow(m.suc) * (x - a)
                mul_assoc((x - a).pow(m.suc), x - a, polynomial_eval(iterated_quotient(p, a, n, m.suc.suc), x))
                (x - a).pow(m.suc) * ((x - a) * polynomial_eval(iterated_quotient(p, a, n, m.suc.suc), x)) =
                    ((x - a).pow(m.suc) * (x - a)) * polynomial_eval(iterated_quotient(p, a, n, m.suc.suc), x)
                distrib((x - a).pow(m.suc), polynomial_eval(qs, a), (x - a) * polynomial_eval(iterated_quotient(p, a, n, m.suc.suc), x))
                (x - a).pow(m.suc) * (polynomial_eval(qs, a) + (x - a) * polynomial_eval(iterated_quotient(p, a, n, m.suc.suc), x)) =
                    (x - a).pow(m.suc) * polynomial_eval(qs, a) + (x - a).pow(m.suc) * ((x - a) * polynomial_eval(iterated_quotient(p, a, n, m.suc.suc), x))
                (x - a).pow(m.suc) * polynomial_eval(qs, x) = (x - a).pow(m.suc) * polynomial_eval(qs, a) + (x - a).pow(m.suc.suc) * polynomial_eval(iterated_quotient(p, a, n, m.suc.suc), x)
                polynomial_eval(qs, a) = polynomial_eval(iterated_quotient(p, a, n, m.suc), a)
                range_sum_suc(function(j: Nat) { polynomial_eval(iterated_quotient(p, a, n, j.suc), a) * (x - a).pow(j.suc) }, m)
                range_sum(function(j: Nat) { polynomial_eval(iterated_quotient(p, a, n, j.suc), a) * (x - a).pow(j.suc) }, m.suc) =
                    range_sum(function(j: Nat) { polynomial_eval(iterated_quotient(p, a, n, j.suc), a) * (x - a).pow(j.suc) }, m) +
                    polynomial_eval(iterated_quotient(p, a, n, m.suc), a) * (x - a).pow(m.suc)
                mul_comm(polynomial_eval(iterated_quotient(p, a, n, m.suc), a), (x - a).pow(m.suc))
                polynomial_eval(iterated_quotient(p, a, n, m.suc), a) * (x - a).pow(m.suc) =
                    (x - a).pow(m.suc) * polynomial_eval(iterated_quotient(p, a, n, m.suc), a)
                polynomial_eval(iterated_quotient(p, a, n, m.suc), a) = polynomial_eval(qs, a)
                polynomial_eval(p, x) = polynomial_eval(p, a) + range_sum(function(j: Nat) { polynomial_eval(iterated_quotient(p, a, n, j.suc), a) * (x - a).pow(j.suc) }, m) + ((x - a).pow(m.suc) * polynomial_eval(qs, a) + (x - a).pow(m.suc.suc) * polynomial_eval(iterated_quotient(p, a, n, m.suc.suc), x))
                polynomial_eval(p, a) + range_sum(function(j: Nat) { polynomial_eval(iterated_quotient(p, a, n, j.suc), a) * (x - a).pow(j.suc) }, m) + ((x - a).pow(m.suc) * polynomial_eval(qs, a) + (x - a).pow(m.suc.suc) * polynomial_eval(iterated_quotient(p, a, n, m.suc.suc), x)) = (polynomial_eval(p, a) + range_sum(function(j: Nat) { polynomial_eval(iterated_quotient(p, a, n, j.suc), a) * (x - a).pow(j.suc) }, m) + (x - a).pow(m.suc) * polynomial_eval(qs, a)) + (x - a).pow(m.suc.suc) * polynomial_eval(iterated_quotient(p, a, n, m.suc.suc), x)
                polynomial_eval(p, x) = (polynomial_eval(p, a) + range_sum(function(j: Nat) { polynomial_eval(iterated_quotient(p, a, n, j.suc), a) * (x - a).pow(j.suc) }, m) + (x - a).pow(m.suc) * polynomial_eval(qs, a)) + (x - a).pow(m.suc.suc) * polynomial_eval(iterated_quotient(p, a, n, m.suc.suc), x)
                polynomial_eval(p, a) + range_sum(function(j: Nat) { polynomial_eval(iterated_quotient(p, a, n, j.suc), a) * (x - a).pow(j.suc) }, m) + (x - a).pow(m.suc) * polynomial_eval(qs, a) = polynomial_eval(p, a) + (range_sum(function(j: Nat) { polynomial_eval(iterated_quotient(p, a, n, j.suc), a) * (x - a).pow(j.suc) }, m) + (x - a).pow(m.suc) * polynomial_eval(qs, a))
                polynomial_eval(p, a) + (range_sum(function(j: Nat) { polynomial_eval(iterated_quotient(p, a, n, j.suc), a) * (x - a).pow(j.suc) }, m) + (x - a).pow(m.suc) * polynomial_eval(qs, a)) = polynomial_eval(p, a) + range_sum(function(j: Nat) { polynomial_eval(iterated_quotient(p, a, n, j.suc), a) * (x - a).pow(j.suc) }, m.suc)
                polynomial_eval(p, x) = polynomial_eval(p, a) + range_sum(function(j: Nat) { polynomial_eval(iterated_quotient(p, a, n, j.suc), a) * (x - a).pow(j.suc) }, m.suc) + (x - a).pow(m.suc.suc) * polynomial_eval(iterated_quotient(p, a, n, m.suc.suc), x)
            }
            st(m.suc)
        }
    }
    st(Nat.0)
    forall(m: Nat) {
        st(m) implies st(m.suc)
    }
    st(Nat.0) and forall(m: Nat) {
        st(m) implies st(m.suc)
    }
    alt_induction(st)
    st(k)
    polynomial_support_bounded_by(p, n) and k <= n
    polynomial_eval(p, x) = polynomial_eval(p, a) + range_sum(function(j: Nat) { polynomial_eval(iterated_quotient(p, a, n, j.suc), a) * (x - a).pow(j.suc) }, k) + (x - a).pow(k.suc) * polynomial_eval(iterated_quotient(p, a, n, k.suc), x)
}

/// The square of the `k`-th power of the primitive `4k`-th root of unity is
/// negative one: `(omega(4k)^k)^2 = -1`.
theorem omega_four_k_sq(k: Nat) {
    k >= Nat.1 implies omega(4 * k).pow(k) * omega(4 * k).pow(k) = -Complex.1
} by {
    if k >= Nat.1 {
        omega(4 * k) = complex_exp(complex_i_mul_real(two * pi / from_nat[Real](4 * k)))
        omega(4 * k).pow(k) = complex_exp(complex_i_mul_real(two * pi / from_nat[Real](4 * k))).pow(k)
        complex_exp_pow(complex_i_mul_real(two * pi / from_nat[Real](4 * k)), k)
        complex_exp(complex_i_mul_real(two * pi / from_nat[Real](4 * k))).pow(k) =
            complex_exp(Complex.from_real(from_nat[Real](k)) * complex_i_mul_real(two * pi / from_nat[Real](4 * k)))
        complex_i_mul_real_mul(from_nat[Real](k), two * pi / from_nat[Real](4 * k))
        Complex.from_real(from_nat[Real](k)) * complex_i_mul_real(two * pi / from_nat[Real](4 * k)) =
            complex_i_mul_real(from_nat[Real](k) * (two * pi / from_nat[Real](4 * k)))
        complex_exp(Complex.from_real(from_nat[Real](k)) * complex_i_mul_real(two * pi / from_nat[Real](4 * k))) =
            complex_exp(complex_i_mul_real(from_nat[Real](k) * (two * pi / from_nat[Real](4 * k))))
        from_nat_mul[Real](Nat.4, k)
        from_nat[Real](Nat.4 * k) = from_nat[Real](Nat.4) * from_nat[Real](k)
        Nat.4 * k = 4 * k
        from_nat[Real](4 * k) = from_nat[Real](Nat.4) * from_nat[Real](k)
        from_nat_two_real
        from_nat[Real](Nat.2) = two
        from_nat_mul[Real](Nat.2, Nat.2)
        from_nat[Real](Nat.2 * Nat.2) = from_nat[Real](Nat.2) * from_nat[Real](Nat.2)
        Nat.2 * Nat.2 = Nat.4
        from_nat[Real](Nat.4) = from_nat[Real](Nat.2) * from_nat[Real](Nat.2)
        from_nat[Real](Nat.4) = two * two
        from_nat[Real](4 * k) = (two * two) * from_nat[Real](k)
        nat_ge_one_ne_zero(k)
        k != Nat.0
        from_nat_suc_pos_real(4 * k - 1)
        from_nat[Real](k) > Real.0
        from_nat[Real](k) != Real.0
        from_nat[Real](k) * (two * pi / from_nat[Real](4 * k)) = from_nat[Real](k) * (two * pi / ((two * two) * from_nat[Real](k)))
        from_nat[Real](k) * (two * pi / ((two * two) * from_nat[Real](k))) = (from_nat[Real](k) * (two * pi)) / ((two * two) * from_nat[Real](k))
        from_nat[Real](k) * (two * pi) = two * pi * from_nat[Real](k)
        (from_nat[Real](k) * (two * pi)) / ((two * two) * from_nat[Real](k)) = (two * pi * from_nat[Real](k)) / ((two * two) * from_nat[Real](k))
        div_cancel_common(two * pi, from_nat[Real](k), two * two)
        from_nat[Real](k) != Real.0 and two * two != Real.0
        two * two != Real.0
        (two * pi * from_nat[Real](k)) / ((two * two) * from_nat[Real](k)) = (two * pi) / (two * two)
        two * pi = pi * two
        (two * pi) / (two * two) = (pi * two) / (two * two)
        div_cancel_common(pi, two, two)
        two != Real.0
        (pi * two) / (two * two) = pi / two
        from_nat[Real](k) * (two * pi / ((two * two) * from_nat[Real](k))) = pi / two
        from_nat[Real](k) * (two * pi / from_nat[Real](4 * k)) = pi / two
        complex_exp(complex_i_mul_real(from_nat[Real](k) * (two * pi / from_nat[Real](4 * k)))) =
            complex_exp(complex_i_mul_real(pi / two))
        omega(4 * k).pow(k) = complex_exp(complex_i_mul_real(pi / two))
        omega(4 * k).pow(k) * omega(4 * k).pow(k) =
            complex_exp(complex_i_mul_real(pi / two)) * complex_exp(complex_i_mul_real(pi / two))
        complex_exp_add(complex_i_mul_real(pi / two), complex_i_mul_real(pi / two))
        complex_exp(complex_i_mul_real(pi / two)) * complex_exp(complex_i_mul_real(pi / two)) =
            complex_exp(complex_i_mul_real(pi / two) + complex_i_mul_real(pi / two))
        complex_i_mul_real_add(pi / two, pi / two)
        complex_i_mul_real(pi / two) + complex_i_mul_real(pi / two) = complex_i_mul_real(pi / two + pi / two)
        pi / two + pi / two = pi
        complex_exp(complex_i_mul_real(pi / two + pi / two)) = complex_exp(complex_i_mul_real(pi))
        complex_exp(complex_i_mul_real(pi / two) + complex_i_mul_real(pi / two)) =
            complex_exp(complex_i_mul_real(pi))
        omega(4 * k).pow(k) * omega(4 * k).pow(k) = complex_exp(complex_i_mul_real(pi))
        euler_identity
        complex_exp(Complex.i * Complex.from_real(pi)) = -Complex.1
        complex_i_mul_real(pi) = Complex.i * Complex.from_real(pi)
        complex_exp(complex_i_mul_real(pi)) = -Complex.1
        omega(4 * k).pow(k) * omega(4 * k).pow(k) = -Complex.1
    }
}

/// The real part of the product with a square root of minus one is a
/// rotation by a quarter turn: `Re((x, y) * w) = -y * Im(w)` when
/// `w = (Re(w), Im(w))` is a square root of `-1`... stated componentwise.
theorem re_mul_im_factor(x: Real, y: Real, u: Real, v: Real) {
    (Complex.new(x, y) * Complex.new(u, v)).re = x * u - y * v
} by {
    re_mul(Complex.new(x, y), Complex.new(u, v))
    (Complex.new(x, y) * Complex.new(u, v)).re =
        Complex.new(x, y).re * Complex.new(u, v).re - Complex.new(x, y).im * Complex.new(u, v).im
    Complex.new(x, y).re = x
    Complex.new(u, v).re = u
    Complex.new(x, y).im = y
    Complex.new(u, v).im = v
    (Complex.new(x, y) * Complex.new(u, v)).re = x * u - y * v
}

/// A complex number with vanishing imaginary part is the embedding of its
/// real part.
theorem complex_eq_from_real_of_im_zero(z: Complex) {
    z.im = Real.0 implies z = Complex.from_real(z.re)
} by {
    if z.im = Real.0 {
        Complex.from_real(z.re) = Complex.new(z.re, Real.0)
        Complex.new(z.re, z.im) = Complex.new(z.re, Real.0)
        z = Complex.new(z.re, z.im)
        complex_eq_new_components(z)
        z = Complex.from_real(z.re)
    }
}

/// A real complex number squares to a nonnegative real.
theorem real_square_real_nonneg(z: Complex) {
    z.im = Real.0 implies (z * z).re >= Real.0
} by {
    if z.im = Real.0 {
        complex_eq_from_real_of_im_zero(z)
        z = Complex.from_real(z.re)
        real_mul_lifts(z.re, z.re)
        Complex.from_real(z.re * z.re) = Complex.from_real(z.re) * Complex.from_real(z.re)
        z * z = Complex.from_real(z.re * z.re)
        re_from_real(z.re * z.re)
        Complex.from_real(z.re * z.re).re = z.re * z.re
        (z * z).re = z.re * z.re
        square_nonneg(z.re)
        z.re * z.re >= Real.0
        (z * z).re >= Real.0
    }
}

/// A real number and its negation cannot both be strictly on one side:
/// `x >= 0` and `-x >= 0` force `x = 0`.
theorem nonneg_and_neg_nonneg_imp_zero(x: Real) {
    x >= Real.0 and -x >= Real.0 implies x = Real.0
} by {
    if x >= Real.0 and -x >= Real.0 {
        lte_add_right(Real.0, -x, x)
        Real.0 <= -x implies Real.0 + x <= -x + x
        Real.0 + x <= -x + x
        Real.0 + x = x
        -x + x = Real.0
        x <= Real.0
        x >= Real.0
        lte_antisymm(x, Real.0)
        x = Real.0
    }
}

/// The imaginary unit is not real-valued: it cannot equal its conjugate.
/// (Used to contradict `Im(w) = 0` for a square root of `-1`.)
theorem im_zero_of_square_neg_one(w: Complex) {
    w * w = -Complex.1 implies not w.im = Real.0
} by {
    if w * w = -Complex.1 {
        if w.im = Real.0 {
            real_square_real_nonneg(w)
            (w * w).re >= Real.0
            neg_re(Complex.1)
            (-Complex.1).re = -(Complex.1.re)
            Complex.1.re = Real.1
            (-Complex.1).re = -Real.1
            w * w = -Complex.1
            (w * w).re = (-Complex.1).re
            (w * w).re = -Real.1
            -Real.1 >= Real.0
            nonneg_and_neg_nonneg_imp_zero(Real.1)
            Real.1 >= Real.0 and -Real.1 >= Real.0 implies Real.1 = Real.0
            real_one_positive
            Real.1 > Real.0
            lt_imp_lte(Real.0, Real.1)
            Real.1 >= Real.0
            Real.1 = Real.0
            false
        }
        not w.im = Real.0
    }
}

/// For a nonzero `d` and a square root `w` of `-1`, one of the four values
/// `Re(d * w^j)`, `j = 0, 1, 2, 3`, is strictly negative.
theorem exists_neg_real_part_four(d: Complex, w: Complex) {
    d != Complex.0 and w * w = -Complex.1
    implies exists(j: Nat) {
        j <= 3 and (d * w.pow(j)).re < Real.0
    }
} by {
    if d != Complex.0 and w * w = -Complex.1 {
        if not exists(j: Nat) { j <= 3 and (d * w.pow(j)).re < Real.0 } {
            forall(t: Nat) {
                if t <= 3 {
                    if (d * w.pow(t)).re < Real.0 {
                        t <= 3 and (d * w.pow(t)).re < Real.0
                        exists(j: Nat) { j <= 3 and (d * w.pow(j)).re < Real.0 }
                        false
                    }
                    (d * w.pow(t)).re >= Real.0
                }
            }
            pow_zero(w)
            w.pow(Nat.0) = Complex.1
            mul_one_right(d)
            d * Complex.1 = d
            (d * w.pow(Nat.0)).re = (d * Complex.1).re
            (d * Complex.1).re = d.re
            (d * w.pow(Nat.0)).re = d.re
            Nat.0 <= 3
            (d * w.pow(Nat.0)).re >= Real.0
            d.re >= Real.0
            complex_pow_suc(w, Nat.1)
            w.pow(Nat.2) = w * w.pow(Nat.1)
            w.pow(Nat.1) = w * w.pow(Nat.0)
            complex_pow_suc(w, Nat.0)
            pow_zero(w)
            w.pow(Nat.0) = Complex.1
            mul_one_right(w)
            w * Complex.1 = w
            w.pow(Nat.1) = w
            w.pow(Nat.2) = w * w
            w * w = -Complex.1
            w.pow(Nat.2) = -Complex.1
            mul_neg_right(d, Complex.1)
            d * -Complex.1 = -(d * Complex.1)
            mul_one_right(d)
            d * Complex.1 = d
            d * -Complex.1 = -d
            (d * w.pow(Nat.2)).re = (d * -Complex.1).re
            (d * -Complex.1).re = (-d).re
            neg_re(d)
            (-d).re = -d.re
            (d * w.pow(Nat.2)).re = -d.re
            lte_suc_suc(Nat.1, Nat.2)
            lte_suc_suc(Nat.0, Nat.1)
            Nat.1 <= 2
            Nat.2 <= 3
            (d * w.pow(Nat.2)).re >= Real.0
            -d.re >= Real.0
            nonneg_and_neg_nonneg_imp_zero(d.re)
            d.re >= Real.0 and -d.re >= Real.0 implies d.re = Real.0
            d.re = Real.0
            w.pow(Nat.3) = w * w.pow(Nat.2)
            complex_pow_suc(w, Nat.2)
            w.pow(Nat.2) = -Complex.1
            w.pow(Nat.3) = w * -Complex.1
            mul_neg_right(w, Complex.1)
            w * -Complex.1 = -(w * Complex.1)
            mul_one_right(w)
            w * Complex.1 = w
            w * -Complex.1 = -w
            w.pow(Nat.3) = -w
            (d * w.pow(Nat.3)).re = (d * -w).re
            mul_neg_left(d, w)
            d * -w = -(d * w)
            (d * -w).re = (-(d * w)).re
            neg_re(d * w)
            (-(d * w)).re = -(d * w).re
            (d * w.pow(Nat.3)).re = -(d * w).re
            Nat.3 <= 3
            (d * w.pow(Nat.3)).re >= Real.0
            -(d * w).re >= Real.0
            nonneg_and_neg_nonneg_imp_zero((d * w).re)
            (d * w).re >= Real.0 and -(d * w).re >= Real.0 implies (d * w).re = Real.0
            lte_suc_suc(Nat.0, Nat.2)
            Nat.0 <= Nat.2
            Nat.1 <= 3
            (d * w.pow(Nat.1)).re >= Real.0
            (d * w).re >= Real.0
            (d * w).re = Real.0
            if d.im = Real.0 {
                d = Complex.new(d.re, d.im)
                complex_eq_new_components(d)
                d.re = Real.0
                d = Complex.new(Real.0, Real.0)
                Complex.from_real(Real.0) = Complex.0
                d = Complex.0
                false
            }
            d.im != Real.0
            im_zero_of_square_neg_one(w)
            not w.im = Real.0
            w.im != Real.0
            re_mul_im_factor(d.re, d.im, w.re, w.im)
            (Complex.new(d.re, d.im) * Complex.new(w.re, w.im)).re = d.re * w.re - d.im * w.im
            d = Complex.new(d.re, d.im)
            w = Complex.new(w.re, w.im)
            complex_eq_new_components(w)
            (d * w).re = d.re * w.re - d.im * w.im
            d.re = Real.0
            (d * w).re = Real.0 * w.re - d.im * w.im
            Real.0 * w.re = Real.0
            (d * w).re = Real.0 - d.im * w.im
            Real.0 - d.im * w.im = -(d.im * w.im)
            (d * w).re = -(d.im * w.im)
            (d * w).re = Real.0
            -(d.im * w.im) = Real.0
            field_mul_nonzero(d.im, w.im)
            d.im != Real.0
            w.im != Real.0
            d.im != Real.0 and w.im != Real.0 implies d.im * w.im != Real.0
            d.im * w.im != Real.0
            if d.im * w.im = Real.0 {
                false
            }
            d.im * w.im != Real.0
            if -(d.im * w.im) = Real.0 {
                neg_neg(d.im * w.im)
                -(-(d.im * w.im)) = d.im * w.im
                -Real.0 = Real.0
                d.im * w.im = Real.0
                false
            }
            false
        }
        exists(j: Nat) { j <= 3 and (d * w.pow(j)).re < Real.0 }
    }
}

/// A support-bounded-by-zero polynomial is the zero polynomial.
theorem polynomial_zero_of_support_bound_zero(p: Polynomial[Complex]) {
    polynomial_support_bounded_by(p, Nat.0) implies p = Polynomial[Complex].zero
} by {
    if polynomial_support_bounded_by(p, Nat.0) {
        polynomial_support_bounded_by(p, Nat.0) = coeff_zero_from(p.coeff, Nat.0)
        coeff_zero_from(p.coeff, Nat.0)
        coeff_zero_from(p.coeff, Nat.0) = forall(k: Nat) {
            coeff_zero_from_at(p.coeff, Nat.0, k)
        }
        forall(k: Nat) {
            coeff_zero_from_at(p.coeff, Nat.0, k)
        }
        forall(k: Nat) {
            coeff_zero_from_at(p.coeff, Nat.0, k) = (not k < Nat.0 implies coeff_zero_at(p.coeff, k))
            not k < Nat.0
            coeff_zero_at(p.coeff, k)
            p.coeff(k) = Complex.0
        }
        forall(k: Nat) { p.coeff(k) = Complex.0 }
        polynomial_eq_zero_of_coeff_zero(p)
        p = Polynomial[Complex].zero
    }
}

/// A range sum with all summands zero is zero.
theorem range_sum_all_zero(f: Nat -> Complex, n: Nat) {
    (forall(i: Nat) { i < n implies f(i) = Complex.0 })
    implies range_sum(f, n) = Complex.0
} by {
    if forall(i: Nat) { i < n implies f(i) = Complex.0 } {
        range_sum_zero_fn[Complex](n)
        range_sum(zero_fn[Complex], n) = Complex.0
        forall(i: Nat) {
            if i < n {
                f(i) = Complex.0
                zero_fn[Complex](i) = Complex.0
                f(i) = zero_fn[Complex](i)
            }
        }
        range_sum_congr(f, zero_fn[Complex], n)
        range_sum(f, n) = range_sum(zero_fn[Complex], n)
        range_sum(f, n) = Complex.0
    }
}

/// A nonconstant polynomial has a first positive index below `n` whose
/// iterated-quotient evaluation at `z0` is nonzero, all earlier ones
/// vanishing.
theorem perturbation_first_coeff(p: Polynomial[Complex], z0: Complex, n: Nat) {
    polynomial_support_bounded_by(p, n) and n >= Nat.1 and polynomial_eval(p, z0) != Complex.0
    and exists(x: Complex, y: Complex) { polynomial_eval(p, x) != polynomial_eval(p, y) }
    implies exists(k: Nat) {
        k >= Nat.1 and k <= n - 1 and
        polynomial_eval(iterated_quotient(p, z0, n, k), z0) != Complex.0 and
        (forall(j: Nat) { j >= Nat.1 and j < k implies polynomial_eval(iterated_quotient(p, z0, n, j), z0) = Complex.0 })
    }
} by {
    if polynomial_support_bounded_by(p, n) and n >= Nat.1 and polynomial_eval(p, z0) != Complex.0
        and exists(x: Complex, y: Complex) { polynomial_eval(p, x) != polynomial_eval(p, y) } {
        define nonzero_coeff(j: Nat) -> Bool {
            j >= Nat.1 and j <= n - 1 and polynomial_eval(iterated_quotient(p, z0, n, j), z0) != Complex.0
        }
        if not exists(j: Nat) { nonzero_coeff(j) } {
            forall(j: Nat) {
                if j >= Nat.1 and j <= n - 1 {
                    if polynomial_eval(iterated_quotient(p, z0, n, j), z0) != Complex.0 {
                        j >= Nat.1 and j <= n - 1 and polynomial_eval(iterated_quotient(p, z0, n, j), z0) != Complex.0
                        nonzero_coeff(j) = (j >= Nat.1 and j <= n - 1 and polynomial_eval(iterated_quotient(p, z0, n, j), z0) != Complex.0)
                        nonzero_coeff(j)
                        exists(t: Nat) { nonzero_coeff(t) }
                        false
                    }
                    polynomial_eval(iterated_quotient(p, z0, n, j), z0) = Complex.0
                }
            }
            polynomial_eval_iterated_expansion(p, z0, z0, n, n - 1)
            n - 1 <= n
            nat_suc_pred_self(n)
            (n - 1).suc = n
            n - 1 < n
            polynomial_support_bounded_by(p, n) and n - 1 <= n
            forall(x: Complex) {
                polynomial_eval_iterated_expansion(p, z0, x, n, n - 1)
                polynomial_support_bounded_by(p, n) and n - 1 <= n
                polynomial_eval(p, x) = polynomial_eval(p, z0) +
                    range_sum(function(j: Nat) { polynomial_eval(iterated_quotient(p, z0, n, j.suc), z0) * (x - z0).pow(j.suc) }, n - 1) + (x - z0).pow(n) * polynomial_eval(iterated_quotient(p, z0, n, n), x)
                forall(t: Nat) {
                    if t < n - 1 {
                        t.suc >= Nat.1
                        t.suc <= n - 1
                        polynomial_eval(iterated_quotient(p, z0, n, t.suc), z0) = Complex.0
                        polynomial_eval(iterated_quotient(p, z0, n, t.suc), z0) * (x - z0).pow(t.suc) = Complex.0 * (x - z0).pow(t.suc)
                        mul_zero_left((x - z0).pow(t.suc))
                        Complex.0 * (x - z0).pow(t.suc) = Complex.0
                        polynomial_eval(iterated_quotient(p, z0, n, t.suc), z0) * (x - z0).pow(t.suc) = Complex.0
                    }
                }
                range_sum_all_zero(function(t: Nat) { polynomial_eval(iterated_quotient(p, z0, n, t.suc), z0) * (x - z0).pow(t.suc) }, n - 1)
                forall(i: Nat) { i < n - 1 implies polynomial_eval(iterated_quotient(p, z0, n, i.suc), z0) * (x - z0).pow(i.suc) = Complex.0 }
                range_sum(function(j: Nat) { polynomial_eval(iterated_quotient(p, z0, n, j.suc), z0) * (x - z0).pow(j.suc) }, n - 1) = Complex.0
                iterated_quotient_support_bounded(p, z0, n, n)
                n <= n
                polynomial_support_bounded_by(iterated_quotient(p, z0, n, n), n - n)
                n - n = Nat.0
                polynomial_support_bounded_by(iterated_quotient(p, z0, n, n), Nat.0)
                polynomial_zero_of_support_bound_zero(iterated_quotient(p, z0, n, n))
                iterated_quotient(p, z0, n, n) = Polynomial[Complex].zero
                polynomial_eval_zero(x)
                polynomial_eval(Polynomial[Complex].zero, x) = Complex.0
                polynomial_eval(iterated_quotient(p, z0, n, n), x) = Complex.0
                (x - z0).pow(n) * polynomial_eval(iterated_quotient(p, z0, n, n), x) = (x - z0).pow(n) * Complex.0
                mul_zero_right((x - z0).pow(n))
                (x - z0).pow(n) * Complex.0 = Complex.0
                polynomial_eval(p, x) = polynomial_eval(p, z0) + Complex.0 + Complex.0
                add_zero_right(polynomial_eval(p, z0))
                polynomial_eval(p, z0) + Complex.0 = polynomial_eval(p, z0)
                polynomial_eval(p, z0) + Complex.0 + Complex.0 = polynomial_eval(p, z0)
                polynomial_eval(p, x) = polynomial_eval(p, z0)
            }
            exists(x: Complex, y: Complex) { polynomial_eval(p, x) != polynomial_eval(p, y) }
            let x0: Complex satisfy {
                exists(y0: Complex) { polynomial_eval(p, x0) != polynomial_eval(p, y0) }
            }
            let y0: Complex satisfy {
                polynomial_eval(p, x0) != polynomial_eval(p, y0)
            }
            polynomial_eval(p, x0) = polynomial_eval(p, z0)
            polynomial_eval(p, y0) = polynomial_eval(p, z0)
            polynomial_eval(p, x0) = polynomial_eval(p, y0)
            false
        }
        exists(j: Nat) { nonzero_coeff(j) }
        exists_least_nat_witness(nonzero_coeff)
        let k: Nat satisfy {
            is_least_nat_witness(nonzero_coeff, k)
        }
        is_least_nat_witness(nonzero_coeff, k) = (nonzero_coeff(k) and forall(j: Nat) { nonzero_coeff(j) implies k <= j })
        nonzero_coeff(k)
        nonzero_coeff(k) = (k >= Nat.1 and k <= n - 1 and polynomial_eval(iterated_quotient(p, z0, n, k), z0) != Complex.0)
        k >= Nat.1
        k <= n - 1
        polynomial_eval(iterated_quotient(p, z0, n, k), z0) != Complex.0
        forall(j: Nat) { nonzero_coeff(j) implies k <= j }
        forall(j: Nat) {
            if j >= Nat.1 and j < k {
                lt_imp_lte(j, k)
                j <= k
                lte_trans(j, k, n - 1)
                j <= n - 1
                if polynomial_eval(iterated_quotient(p, z0, n, j), z0) != Complex.0 {
                    j >= Nat.1 and j <= n - 1 and polynomial_eval(iterated_quotient(p, z0, n, j), z0) != Complex.0
                    nonzero_coeff(j) = (j >= Nat.1 and j <= n - 1 and polynomial_eval(iterated_quotient(p, z0, n, j), z0) != Complex.0)
                    nonzero_coeff(j)
                    nonzero_coeff(j) implies k <= j
                    k <= j
                    j < k
                    false
                }
                polynomial_eval(iterated_quotient(p, z0, n, j), z0) = Complex.0
            }
        }
        k >= Nat.1 and k <= n - 1 and
            polynomial_eval(iterated_quotient(p, z0, n, k), z0) != Complex.0 and
            (forall(j: Nat) { j >= Nat.1 and j < k implies polynomial_eval(iterated_quotient(p, z0, n, j), z0) = Complex.0 })
        exists(kt: Nat) {
            kt >= Nat.1 and kt <= n - 1 and
            polynomial_eval(iterated_quotient(p, z0, n, kt), z0) != Complex.0 and
            (forall(j: Nat) { j >= Nat.1 and j < kt implies polynomial_eval(iterated_quotient(p, z0, n, j), z0) = Complex.0 })
        }
    }
}

/// The perturbed evaluation around a first nonzero shifted coefficient:
/// `p(z0 + w) = p(z0) + c_k * w^k + w^(k+1) * q_{k+1}(z0 + w)`.
theorem perturbation_expansion(p: Polynomial[Complex], z0: Complex, w: Complex, n: Nat, k: Nat) {
    polynomial_support_bounded_by(p, n) and n >= Nat.1 and k >= Nat.1 and k <= n - 1
    and polynomial_eval(iterated_quotient(p, z0, n, k), z0) != Complex.0
    and (forall(j: Nat) { j >= Nat.1 and j < k implies polynomial_eval(iterated_quotient(p, z0, n, j), z0) = Complex.0 })
    implies polynomial_eval(p, z0 + w) = polynomial_eval(p, z0) +
        polynomial_eval(iterated_quotient(p, z0, n, k), z0) * w.pow(k) +
        w.pow(k.suc) * polynomial_eval(iterated_quotient(p, z0, n, k.suc), z0 + w)
} by {
    if polynomial_support_bounded_by(p, n) and n >= Nat.1 and k >= Nat.1 and k <= n - 1
        and polynomial_eval(iterated_quotient(p, z0, n, k), z0) != Complex.0
        and (forall(j: Nat) { j >= Nat.1 and j < k implies polynomial_eval(iterated_quotient(p, z0, n, j), z0) = Complex.0 }) {
        nat_pred_lt_self(n)
        n - 1 < n
        lt_imp_lte(n - 1, n)
        n - 1 <= n
        lte_trans(k, n - 1, n)
        k <= n
        polynomial_eval_iterated_expansion(p, z0, z0 + w, n, k)
        polynomial_support_bounded_by(p, n) and k <= n
        polynomial_eval(p, z0 + w) = polynomial_eval(p, z0) +
            range_sum(function(j: Nat) { polynomial_eval(iterated_quotient(p, z0, n, j.suc), z0) * ((z0 + w) - z0).pow(j.suc) }, k) + ((z0 + w) - z0).pow(k.suc) * polynomial_eval(iterated_quotient(p, z0, n, k.suc), z0 + w)
        (z0 + w) - z0 = w
        nat_suc_pred_self(k)
        (k - 1).suc = k
        range_sum_suc(function(j: Nat) { polynomial_eval(iterated_quotient(p, z0, n, j.suc), z0) * w.pow(j.suc) }, k - 1)
        range_sum(function(j: Nat) { polynomial_eval(iterated_quotient(p, z0, n, j.suc), z0) * w.pow(j.suc) }, k) =
            range_sum(function(j: Nat) { polynomial_eval(iterated_quotient(p, z0, n, j.suc), z0) * w.pow(j.suc) }, k - 1) +
            polynomial_eval(iterated_quotient(p, z0, n, k), z0) * w.pow(k)
        forall(t: Nat) {
            if t < k - 1 {
                t.suc >= Nat.1
                lt_imp_lte_suc(t, k - 1)
                t.suc <= k - 1
                nat_pred_lt_self(k)
                k - 1 < k
                lte_lt_trans(t.suc, k - 1, k)
                t.suc < k
                polynomial_eval(iterated_quotient(p, z0, n, t.suc), z0) = Complex.0
                polynomial_eval(iterated_quotient(p, z0, n, t.suc), z0) * w.pow(t.suc) = Complex.0 * w.pow(t.suc)
                mul_zero_left(w.pow(t.suc))
                Complex.0 * w.pow(t.suc) = Complex.0
                polynomial_eval(iterated_quotient(p, z0, n, t.suc), z0) * w.pow(t.suc) = Complex.0
            }
        }
        range_sum_all_zero(function(j: Nat) { polynomial_eval(iterated_quotient(p, z0, n, j.suc), z0) * w.pow(j.suc) }, k - 1)
        forall(i: Nat) { i < k - 1 implies polynomial_eval(iterated_quotient(p, z0, n, i.suc), z0) * w.pow(i.suc) = Complex.0 }
        range_sum(function(j: Nat) { polynomial_eval(iterated_quotient(p, z0, n, j.suc), z0) * w.pow(j.suc) }, k - 1) = Complex.0
        range_sum(function(j: Nat) { polynomial_eval(iterated_quotient(p, z0, n, j.suc), z0) * w.pow(j.suc) }, k) =
            Complex.0 + polynomial_eval(iterated_quotient(p, z0, n, k), z0) * w.pow(k)
        add_zero_left(polynomial_eval(iterated_quotient(p, z0, n, k), z0) * w.pow(k))
        Complex.0 + polynomial_eval(iterated_quotient(p, z0, n, k), z0) * w.pow(k) =
            polynomial_eval(iterated_quotient(p, z0, n, k), z0) * w.pow(k)
        range_sum(function(j: Nat) { polynomial_eval(iterated_quotient(p, z0, n, j.suc), z0) * w.pow(j.suc) }, k) =
            polynomial_eval(iterated_quotient(p, z0, n, k), z0) * w.pow(k)
        polynomial_eval(p, z0 + w) = polynomial_eval(p, z0) + range_sum(function(j: Nat) { polynomial_eval(iterated_quotient(p, z0, n, j.suc), z0) * ((z0 + w) - z0).pow(j.suc) }, k) + ((z0 + w) - z0).pow(k.suc) * polynomial_eval(iterated_quotient(p, z0, n, k.suc), z0 + w)
        (z0 + w) - z0 = w
        forall(t: Nat) {
            if t < k {
                polynomial_eval(iterated_quotient(p, z0, n, t.suc), z0) * ((z0 + w) - z0).pow(t.suc) = polynomial_eval(iterated_quotient(p, z0, n, t.suc), z0) * w.pow(t.suc)
            }
        }
        range_sum_congr(function(j: Nat) { polynomial_eval(iterated_quotient(p, z0, n, j.suc), z0) * ((z0 + w) - z0).pow(j.suc) },
            function(j: Nat) { polynomial_eval(iterated_quotient(p, z0, n, j.suc), z0) * w.pow(j.suc) }, k)
        range_sum(function(j: Nat) { polynomial_eval(iterated_quotient(p, z0, n, j.suc), z0) * ((z0 + w) - z0).pow(j.suc) }, k) =
            range_sum(function(j: Nat) { polynomial_eval(iterated_quotient(p, z0, n, j.suc), z0) * w.pow(j.suc) }, k)
        polynomial_eval(p, z0 + w) = polynomial_eval(p, z0) + range_sum(function(j: Nat) { polynomial_eval(iterated_quotient(p, z0, n, j.suc), z0) * w.pow(j.suc) }, k) + ((z0 + w) - z0).pow(k.suc) * polynomial_eval(iterated_quotient(p, z0, n, k.suc), z0 + w)
        polynomial_eval(p, z0 + w) = polynomial_eval(p, z0) + (polynomial_eval(iterated_quotient(p, z0, n, k), z0) * w.pow(k)) + ((z0 + w) - z0).pow(k.suc) * polynomial_eval(iterated_quotient(p, z0, n, k.suc), z0 + w)
        (z0 + w) - z0 = w
        ((z0 + w) - z0).pow(k.suc) = w.pow(k.suc)
        polynomial_eval(p, z0 + w) = polynomial_eval(p, z0) + polynomial_eval(iterated_quotient(p, z0, n, k), z0) * w.pow(k) + w.pow(k.suc) * polynomial_eval(iterated_quotient(p, z0, n, k.suc), z0 + w)
    }
}

/// A power of a number in `[0, 1]` is at most the number itself.
theorem real_pow_le_self(t: Real, k: Nat) {
    t >= Real.0 and t <= Real.1 and k >= Nat.1 implies t.pow(k) <= t
} by {
    if t >= Real.0 and t <= Real.1 and k >= Nat.1 {
        real_pow_le_one(t, k - 1)
        t >= Real.0 and t <= Real.1 implies t.pow(k - 1) <= Real.1
        t.pow(k - 1) <= Real.1
        mul_le_mul_of_nonneg_right(t.pow(k - 1), Real.1, t)
        t.pow(k - 1) <= Real.1 and t >= Real.0 implies t.pow(k - 1) * t <= Real.1 * t
        t.pow(k - 1) * t <= Real.1 * t
        Real.1 * t = t
        t.pow(k - 1) * t <= t
        nat_suc_pred_self(k)
        (k - 1).suc = k
        real_pow_suc(t, k - 1)
        t.pow(k) = t.pow(k - 1) * t
        t.pow(k) <= t
    }
}

/// A positive `a` times its reciprocal of a product cancels:
/// `two * a * (x / (two * a)) = x` for `a != 0`.
theorem real_mul_div_cancel_two(a: Real, x: Real) {
    a != Real.0 implies two * a * (x / (two * a)) = x
} by {
    if a != Real.0 {
        two * a != Real.0
        x / (two * a) = x * (two * a).inverse
        two * a * (x / (two * a)) = two * a * (x * (two * a).inverse)
        two * a * (x * (two * a).inverse) = x * ((two * a) * (two * a).inverse)
        (two * a) * (two * a).inverse = Real.1
        x * ((two * a) * (two * a).inverse) = x * Real.1
        x * Real.1 = x
        two * a * (x / (two * a)) = x
    }
}

/// The square of a difference with a divided term expands.
theorem real_square_sub_div(a: Real, x: Real) {
    (a - x / (two * a)) * (a - x / (two * a)) =
        a * a - two * a * (x / (two * a)) + (x / (two * a)) * (x / (two * a))
} by {
    let y: Real = x / (two * a)
    mul_sub_distrib_right(a, y, a - y)
    (a - y) * (a - y) = (a - y) * a - (a - y) * y
    mul_sub_distrib_left(a, a, y)
    (a - y) * a = a * a - y * a
    (a - y) * y = a * y - y * y
    (a - y) * (a - y) = (a * a - y * a) - (a * y - y * y)
    (a * a - y * a) - (a * y - y * y) = (a * a - y * a) + -(a * y - y * y)
    a * y - y * y = a * y + -(y * y)
    -(a * y - y * y) = -(a * y) + -(-(y * y))
    -(-(y * y)) = y * y
    -(a * y - y * y) = -(a * y) + y * y
    (a * a - y * a) + (-(a * y) + y * y) = a * a - y * a - a * y + y * y
    (a * a - y * a) - (a * y - y * y) = a * a - y * a - a * y + y * y
    y * a = a * y
    real_mul_comm(y, a)
    a * a - y * a - a * y + y * y = a * a - a * y - a * y + y * y
    a * y + a * y = two * (a * y)
    a * a - a * y - a * y + y * y = a * a - two * (a * y) + y * y
    a * a - two * (a * y) + y * y = a * a - two * a * y + y * y
    (a - y) * (a - y) = a * a - two * a * y + y * y
    y = x / (two * a)
    (a - x / (two * a)) * (a - x / (two * a)) = a * a - two * a * (x / (two * a)) + (x / (two * a)) * (x / (two * a))
}

/// If a nonnegative square is at most `a^2 - x`, the number is at most
/// `a - x / (2a)`.
theorem modulus_square_bound(m: Complex, a: Real, x: Real) {
    a > Real.0 and x >= Real.0 and x <= a * a / two and (a * a - x) = m.modulus * m.modulus
    implies m.modulus <= a - x / (two * a)
} by {
    if a > Real.0 and x >= Real.0 and x <= a * a / two and (a * a - x) = m.modulus * m.modulus {
        a >= Real.0
        modulus_nonneg(m)
        m.modulus >= Real.0
        real_square_sub_div(a, x)
        (a - x / (two * a)) * (a - x / (two * a)) = a * a - two * a * (x / (two * a)) + (x / (two * a)) * (x / (two * a))
        a != Real.0
        real_mul_div_cancel_two(a, x)
        two * a * (x / (two * a)) = x
        a * a - two * a * (x / (two * a)) + (x / (two * a)) * (x / (two * a)) = a * a - x + (x / (two * a)) * (x / (two * a))
        (a - x / (two * a)) * (a - x / (two * a)) = a * a - x + (x / (two * a)) * (x / (two * a))
        square_nonneg(x / (two * a))
        (x / (two * a)) * (x / (two * a)) >= Real.0
        add_lte_add(a * a - x, a * a - x, Real.0, (x / (two * a)) * (x / (two * a)))
        a * a - x <= a * a - x and Real.0 <= (x / (two * a)) * (x / (two * a)) implies a * a - x + Real.0 <= a * a - x + (x / (two * a)) * (x / (two * a))
        lte_self(a * a - x)
        a * a - x <= a * a - x
        a * a - x + Real.0 = a * a - x
        a * a - x = a * a - x + Real.0
        a * a - x <= a * a - x + Real.0
        lte_trans(a * a - x, a * a - x + Real.0, a * a - x + (x / (two * a)) * (x / (two * a)))
        a * a - x <= a * a - x + (x / (two * a)) * (x / (two * a))
        lte_trans(a * a - x, a * a - x + (x / (two * a)) * (x / (two * a)), (a - x / (two * a)) * (a - x / (two * a)))
        a * a - x <= (a - x / (two * a)) * (a - x / (two * a))
        m.modulus * m.modulus <= (a - x / (two * a)) * (a - x / (two * a))
        x <= a * a / two
        square_nonneg(a)
        a * a >= Real.0
        mul_div_cancel(a * a, two)
        two != Real.0
        two * (a * a / two) = a * a
        two_mul_eq_add_self_real(a * a / two)
        two * (a * a / two) = a * a / two + a * a / two
        a * a = a * a / two + a * a / two
        lte_self(a * a / two)
        a * a / two <= a * a / two
        real_two_positive
        two > Real.0
        real_inv_pos(two)
        two.inverse > Real.0
        two.inverse >= Real.0
        a * a / two = (a * a) * two.inverse
        mul_nonneg(a * a, two.inverse)
        a * a >= Real.0 and two.inverse >= Real.0 implies (a * a) * two.inverse >= Real.0
        a * a >= Real.0
        (a * a) * two.inverse >= Real.0
        a * a / two >= Real.0
        add_lte_add(a * a / two, a * a / two, Real.0, a * a / two)
        a * a / two <= a * a / two and Real.0 <= a * a / two implies a * a / two + Real.0 <= a * a / two + a * a / two
        a * a / two + Real.0 = a * a / two
        a * a / two + Real.0 <= a * a / two + a * a / two
        a * a / two <= a * a / two + a * a / two
        a * a / two + a * a / two <= a * a
        lte_trans(a * a / two, a * a / two + a * a / two, a * a)
        a * a / two <= a * a
        lte_trans(x, a * a / two, a * a)
        x <= a * a
        two_mul_eq_add_self_real(a * a)
        two * (a * a) = a * a + a * a
        lte_self(a * a)
        a * a <= a * a
        add_lte_add(a * a, a * a, Real.0, a * a)
        a * a <= a * a and Real.0 <= a * a implies a * a + Real.0 <= a * a + a * a
        a * a + Real.0 = a * a
        a * a <= a * a + a * a
        a * a <= two * (a * a)
        lte_trans(x, a * a, two * (a * a))
        x <= two * (a * a)
        real_two_positive
        two > Real.0
        two.is_positive
        a > Real.0
        a.is_positive
        mul_pos_pos(two, a)
        two.is_positive and a.is_positive implies (two * a).is_positive
        (two * a).is_positive
        two * a > Real.0
        (two * a).inverse > Real.0
        (two * a).inverse >= Real.0
        mul_le_mul_of_nonneg_right(x, two * (a * a), (two * a).inverse)
        x <= two * (a * a) and (two * a).inverse >= Real.0 implies x * (two * a).inverse <= two * (a * a) * (two * a).inverse
        x * (two * a).inverse <= two * (a * a) * (two * a).inverse
        x / (two * a) = x * (two * a).inverse
        two * (a * a) / (two * a) = two * (a * a) * (two * a).inverse
        x / (two * a) <= two * (a * a) / (two * a)
        div_cancel_common(a, two * a, Real.1)
        two * a != Real.0
        Real.1 != Real.0
        (a * (two * a)) / (Real.1 * (two * a)) = a / Real.1
        a * (two * a) = two * (a * a)
        Real.1 * (two * a) = two * a
        two * (a * a) / (two * a) = a / Real.1
        a / Real.1 = a
        two * (a * a) / (two * a) = a
        lte_trans(x / (two * a), two * (a * a) / (two * a), a)
        x / (two * a) <= a
        lte_add_right(x / (two * a), a, -(x / (two * a)))
        x / (two * a) + -(x / (two * a)) <= a + -(x / (two * a))
        x / (two * a) + -(x / (two * a)) = Real.0
        a + -(x / (two * a)) = a - x / (two * a)
        Real.0 <= a - x / (two * a)
        a - x / (two * a) >= Real.0
        square_le_square_of_nonneg(m.modulus, a - x / (two * a))
        m.modulus <= a - x / (two * a)
    }
}

/// The real part of a real scalar times a complex number.
theorem re_scalar_mul(r: Real, z: Complex) {
    (Complex.from_real(r) * z).re = r * z.re
} by {
    re_mul(Complex.from_real(r), z)
    (Complex.from_real(r) * z).re =
        Complex.from_real(r).re * z.re - Complex.from_real(r).im * z.im
    re_from_real(r)
    Complex.from_real(r).re = r
    Complex.from_real(r).im = Real.0
    (Complex.from_real(r) * z).re = r * z.re - Real.0 * z.im
    Real.0 * z.im = Real.0
    r * z.re - Real.0 = r * z.re
    (Complex.from_real(r) * z).re = r * z.re
}

/// Conjugation preserves the real part.
theorem re_conj_local(z: Complex) {
    z.conj.re = z.re
} by {
    re_conj(z)
    z.conj.re = z.re
}

/// The real part of a product with a conjugate equals the real part of the
/// conjugated product: `Re(A * conj(c * w^k)) = Re(conj(A) * c * w^k)`.
theorem re_mul_conj_pow(a: Complex, c: Complex, w: Complex, k: Nat) {
    (a * (c * w.pow(k)).conj).re = (a.conj * c * w.pow(k)).re
} by {
    conj_mul(c, w.pow(k))
    (c * w.pow(k)).conj = c.conj * (w.pow(k)).conj
    a * (c * w.pow(k)).conj = a * (c.conj * (w.pow(k)).conj)
    conj_mul(a, c.conj * (w.pow(k)).conj)
    (a * (c.conj * (w.pow(k)).conj)).conj = a.conj * (c.conj * (w.pow(k)).conj).conj
    conj_mul(c.conj, (w.pow(k)).conj)
    (c.conj * (w.pow(k)).conj).conj = (c.conj).conj * ((w.pow(k)).conj).conj
    conj_conj(c)
    (c.conj).conj = c
    conj_conj(w.pow(k))
    ((w.pow(k)).conj).conj = w.pow(k)
    (c.conj * (w.pow(k)).conj).conj = c * w.pow(k)
    (a * (c.conj * (w.pow(k)).conj)).conj = a.conj * (c * w.pow(k))
    a * (c * w.pow(k)).conj = (a.conj * (c * w.pow(k))).conj
    re_conj_local(a.conj * (c * w.pow(k)))
    (a.conj * (c * w.pow(k))).conj.re = (a.conj * (c * w.pow(k))).re
    (a * (c * w.pow(k)).conj).re = (a.conj * (c * w.pow(k))).re
    a.conj * c * w.pow(k) = a.conj * (c * w.pow(k))
    mul_assoc(a.conj, c, w.pow(k))
    (a * (c * w.pow(k)).conj).re = (a.conj * c * w.pow(k)).re
}

/// A nonzero complex number has a nonzero conjugate.
theorem conj_ne_zero(z: Complex) {
    z != Complex.0 implies z.conj != Complex.0
} by {
    if z != Complex.0 {
        if z.conj = Complex.0 {
            conj_zero
            z.conj = Complex.0
            conj_conj(z)
            (z.conj).conj = z
            z.conj = Complex.0
            z = Complex.0
            false
        }
        z.conj != Complex.0
    }
}

/// The modulus of a power of a unit is one.
theorem modulus_unit_pow(w: Complex, j: Nat) {
    w.modulus = Real.1 implies (w.pow(j)).modulus = Real.1
} by {
    if w.modulus = Real.1 {
        complex_modulus_pow(w, j)
        w.pow(j).modulus = w.modulus.pow(j)
        one_pow[Real](j)
        Real.1.pow(j) = Real.1
        w.pow(j).modulus = Real.1
    }
}

/// The modulus of `omega(4k)` is one.
theorem omega_four_k_unit_modulus(k: Nat) {
    k >= Nat.1 implies omega(4 * k).modulus = Real.1
} by {
    if k >= Nat.1 {
        nat_ge_one_ne_zero(k)
        k != Nat.0
        zero_or_suc(k)
        let m: Nat satisfy {
            k = m.suc
        }
        k = m.suc
        4 * k = 4 * m.suc
        4 * m.suc = 4 * m + 4
        4 * m + 4 = (4 * m + 3).suc
        0 <= 4 * m + 3
        lte_suc_suc(0, 4 * m + 3)
        1 <= (4 * m + 3).suc
        1 <= 4 * m + 4
        4 * m + 4 >= 1
        4 * k >= 1
        omega_modulus(4 * k)
        4 * k >= Nat.1
        omega(4 * k).modulus = Real.1
    }
}

/// The inverse of a product is the product of the inverses.
theorem real_inverse_mul(a: Real, b: Real) {
    a != Real.0 and b != Real.0 implies (a * b).inverse = a.inverse * b.inverse
} by {
    if a != Real.0 and b != Real.0 {
        a * b != Real.0
        (a * b) * ((a * b).inverse) = Real.1
        (a * b) * (a.inverse * b.inverse) = a * (b * (a.inverse * b.inverse))
        a * (b * (a.inverse * b.inverse)) = a * ((b * a.inverse) * b.inverse)
        real_mul_comm(b, a.inverse)
        b * a.inverse = a.inverse * b
        a * ((b * a.inverse) * b.inverse) = a * ((a.inverse * b) * b.inverse)
        a * ((a.inverse * b) * b.inverse) = a * (a.inverse * (b * b.inverse))
        a * (a.inverse * (b * b.inverse)) = (a * a.inverse) * (b * b.inverse)
        (a * b) * (a.inverse * b.inverse) = (a * a.inverse) * (b * b.inverse)
        a * a.inverse = Real.1
        b * b.inverse = Real.1
        (a * a.inverse) * (b * b.inverse) = Real.1 * Real.1
        Real.1 * Real.1 = Real.1
        (a * b) * (a.inverse * b.inverse) = Real.1
        (a * b) * (a.inverse * b.inverse) = (a * b) * ((a * b).inverse)
        left_cancel(a * b, a.inverse * b.inverse, (a * b).inverse)
        (a * b) * (a.inverse * b.inverse) = (a * b) * ((a * b).inverse) implies a.inverse * b.inverse = (a * b).inverse
        a.inverse * b.inverse = (a * b).inverse
        (a * b).inverse = a.inverse * b.inverse
    }
}

/// A small perturbation size with the required bounds exists.
theorem perturbation_t_choice(rho: Real, b: Real, aa: Real, hh: Real) {
    rho > Real.0 and b > Real.0 and aa > Real.0 and hh >= Real.0
    implies exists(t: Real) {
        t > Real.0 and t <= Real.1 and t <= rho / (b * b) and t <= aa * aa / (real_four * rho) and t <= rho / (real_four * aa * (hh + Real.1))
    }
} by {
    if rho > Real.0 and b > Real.0 and aa > Real.0 and hh >= Real.0 {
        rho.is_positive
        b.is_positive
        mul_pos_pos(b, b)
        b.is_positive and b.is_positive implies (b * b).is_positive
        (b * b).is_positive
        b * b > Real.0
        b * b != Real.0
        real_inv_pos(b * b)
        (b * b).inverse > Real.0
        (b * b).inverse.is_positive
        rho / (b * b) = rho * (b * b).inverse
        mul_pos_pos(rho, (b * b).inverse)
        rho.is_positive and (b * b).inverse.is_positive implies (rho * (b * b).inverse).is_positive
        (rho * (b * b).inverse).is_positive
        rho / (b * b) > Real.0
        (rho / (b * b)).is_positive
        aa.is_positive
        mul_pos_pos(aa, aa)
        aa.is_positive and aa.is_positive implies (aa * aa).is_positive
        (aa * aa).is_positive
        aa * aa > Real.0
        real_two_positive
        two > Real.0
        lte_self(two)
        two <= two
        two >= Real.0
        add_lte_add(two, two, Real.0, two)
        two <= two and Real.0 <= two implies two + Real.0 <= two + two
        two + Real.0 = two
        two <= two + two
        two + two >= two
        lt_lte_trans(Real.0, two, two + two)
        two + two > Real.0
        real_four = two + two
        real_four > Real.0
        real_four.is_positive
        mul_pos_pos(real_four, rho)
        real_four.is_positive and rho.is_positive implies (real_four * rho).is_positive
        (real_four * rho).is_positive
        real_four * rho > Real.0
        real_four * rho != Real.0
        real_inv_pos(real_four * rho)
        (real_four * rho).inverse > Real.0
        (real_four * rho).inverse.is_positive
        aa * aa / (real_four * rho) = aa * aa * (real_four * rho).inverse
        mul_pos_pos(aa * aa, (real_four * rho).inverse)
        (aa * aa).is_positive and (real_four * rho).inverse.is_positive implies ((aa * aa) * (real_four * rho).inverse).is_positive
        ((aa * aa) * (real_four * rho).inverse).is_positive
        aa * aa / (real_four * rho) > Real.0
        (aa * aa / (real_four * rho)).is_positive
        lte_add_right(Real.0, hh, Real.1)
        Real.0 <= hh implies Real.0 + Real.1 <= hh + Real.1
        Real.0 + Real.1 = Real.1
        Real.1 <= hh + Real.1
        real_one_positive
        Real.1 > Real.0
        lt_imp_lte(Real.0, Real.1)
        Real.0 <= Real.1
        lte_trans(Real.0, Real.1, hh + Real.1)
        Real.0 <= hh + Real.1
        lt_lte_trans(Real.0, Real.1, hh + Real.1)
        Real.0 < Real.1 and Real.1 <= hh + Real.1 implies Real.0 < hh + Real.1
        Real.0 < Real.1
        Real.1 <= hh + Real.1
        Real.0 < hh + Real.1
        hh + Real.1 > Real.0
        (hh + Real.1).is_positive
        real_four > Real.0
        aa > Real.0
        real_four.is_positive
        aa.is_positive
        mul_pos_pos(real_four, aa)
        real_four.is_positive and aa.is_positive implies (real_four * aa).is_positive
        (real_four * aa).is_positive
        real_four * aa > Real.0
        real_four * aa != Real.0
        mul_pos_pos(real_four * aa, hh + Real.1)
        (real_four * aa).is_positive and (hh + Real.1).is_positive implies ((real_four * aa) * (hh + Real.1)).is_positive
        ((real_four * aa) * (hh + Real.1)).is_positive
        real_four * aa * (hh + Real.1) > Real.0
        real_four * aa * (hh + Real.1) != Real.0
        real_inv_pos(real_four * aa * (hh + Real.1))
        (real_four * aa * (hh + Real.1)).inverse > Real.0
        (real_four * aa * (hh + Real.1)).inverse.is_positive
        rho / (real_four * aa * (hh + Real.1)) = rho * (real_four * aa * (hh + Real.1)).inverse
        mul_pos_pos(rho, (real_four * aa * (hh + Real.1)).inverse)
        rho.is_positive and (real_four * aa * (hh + Real.1)).inverse.is_positive implies (rho * (real_four * aa * (hh + Real.1)).inverse).is_positive
        (rho * (real_four * aa * (hh + Real.1)).inverse).is_positive
        rho / (real_four * aa * (hh + Real.1)) > Real.0
        (rho / (real_four * aa * (hh + Real.1))).is_positive
        let t: Real = (((Real.1).min(rho / (b * b))).min(aa * aa / (real_four * rho))).min(rho / (real_four * aa * (hh + Real.1)))
        min_pos_pos(Real.1, rho / (b * b))
        Real.1 > Real.0
        Real.1.is_positive
        rho / (b * b) > Real.0
        (rho / (b * b)).is_positive
        Real.1.is_positive and (rho / (b * b)).is_positive
        Real.1.min(rho / (b * b)).is_positive
        min_pos_pos(Real.1.min(rho / (b * b)), aa * aa / (real_four * rho))
        Real.1.min(rho / (b * b)).is_positive and (aa * aa / (real_four * rho)).is_positive
        Real.1.min(rho / (b * b)).min(aa * aa / (real_four * rho)).is_positive
        min_pos_pos(Real.1.min(rho / (b * b)).min(aa * aa / (real_four * rho)), rho / (real_four * aa * (hh + Real.1)))
        Real.1.min(rho / (b * b)).min(aa * aa / (real_four * rho)).is_positive and (rho / (real_four * aa * (hh + Real.1))).is_positive
        t.is_positive
        t > Real.0
        min_lte_left(Real.1, rho / (b * b))
        Real.1.min(rho / (b * b)) <= Real.1
        min_lte_left(Real.1.min(rho / (b * b)), aa * aa / (real_four * rho))
        Real.1.min(rho / (b * b)).min(aa * aa / (real_four * rho)) <= Real.1.min(rho / (b * b))
        min_lte_left(Real.1.min(rho / (b * b)).min(aa * aa / (real_four * rho)), rho / (real_four * aa * (hh + Real.1)))
        t <= Real.1.min(rho / (b * b)).min(aa * aa / (real_four * rho))
        lte_trans(t, Real.1.min(rho / (b * b)).min(aa * aa / (real_four * rho)), Real.1.min(rho / (b * b)))
        lte_trans(t, Real.1.min(rho / (b * b)), Real.1)
        t <= Real.1
        min_lte_right(Real.1, rho / (b * b))
        Real.1.min(rho / (b * b)) <= rho / (b * b)
        lte_trans(t, Real.1.min(rho / (b * b)), rho / (b * b))
        t <= rho / (b * b)
        min_lte_right(Real.1.min(rho / (b * b)), aa * aa / (real_four * rho))
        Real.1.min(rho / (b * b)).min(aa * aa / (real_four * rho)) <= aa * aa / (real_four * rho)
        lte_trans(t, Real.1.min(rho / (b * b)).min(aa * aa / (real_four * rho)), aa * aa / (real_four * rho))
        t <= aa * aa / (real_four * rho)
        min_lte_right(Real.1.min(rho / (b * b)).min(aa * aa / (real_four * rho)), rho / (real_four * aa * (hh + Real.1)))
        t <= rho / (real_four * aa * (hh + Real.1))
        t > Real.0 and t <= Real.1 and t <= rho / (b * b) and t <= aa * aa / (real_four * rho) and t <= rho / (real_four * aa * (hh + Real.1))
        exists(t0: Real) {
            t0 > Real.0 and t0 <= Real.1 and t0 <= rho / (b * b) and t0 <= aa * aa / (real_four * rho) and t0 <= rho / (real_four * aa * (hh + Real.1))
        }
    }
}

/// Four is positive.
theorem real_four_pos {
    real_four > Real.0
} by {
    real_four = two + two
    lte_self(two)
    two <= two
    two >= Real.0
    add_lte_add(two, two, Real.0, two)
    two <= two and Real.0 <= two implies two + Real.0 <= two + two
    two + Real.0 = two
    two <= two + two
    real_two_positive
    two > Real.0
    lt_lte_trans(Real.0, two, two + two)
    two + two > Real.0
    real_four > Real.0
}

/// A quotient of positive reals is positive.
theorem real_div_pos(a: Real, b: Real) {
    a > Real.0 and b > Real.0 implies a / b > Real.0
} by {
    if a > Real.0 and b > Real.0 {
        a.is_positive
        b != Real.0
        real_inv_pos(b)
        b.inverse > Real.0
        b.inverse.is_positive
        a / b = a * b.inverse
        mul_pos_pos(a, b.inverse)
        a.is_positive and b.inverse.is_positive implies (a * b.inverse).is_positive
        (a * b.inverse).is_positive
        a * b.inverse > Real.0
        a / b > Real.0
    }
}

/// A sum with a nonnegative first part and a positive part is positive:
/// `hh + 1 > 0` for `hh >= 0`.
theorem real_plus_one_pos(hh: Real) {
    hh >= Real.0 implies hh + Real.1 > Real.0
} by {
    if hh >= Real.0 {
        lte_add_right(Real.0, hh, Real.1)
        Real.0 <= hh implies Real.0 + Real.1 <= hh + Real.1
        Real.0 + Real.1 = Real.1
        Real.1 <= hh + Real.1
        real_one_positive
        Real.1 > Real.0
        lt_imp_lte(Real.0, Real.1)
        Real.0 <= Real.1
        lte_trans(Real.0, Real.1, hh + Real.1)
        Real.0 <= hh + Real.1
        lt_lte_trans(Real.0, Real.1, hh + Real.1)
        Real.0 < Real.1 and Real.1 <= hh + Real.1 implies Real.0 < hh + Real.1
        Real.0 < Real.1
        Real.1 <= hh + Real.1
        Real.0 < hh + Real.1
        hh + Real.1 > Real.0
    }
}

/// A product of positive reals is nonzero.
theorem real_mul_pos_ne_zero(a: Real, b: Real) {
    a > Real.0 and b > Real.0 implies a * b != Real.0
} by {
    if a > Real.0 and b > Real.0 {
        mul_pos_pos(a, b)
        a.is_positive and b.is_positive implies (a * b).is_positive
        a.is_positive
        b.is_positive
        (a * b).is_positive
        (a * b) != Real.0
    }
}

/// A negative real has a positive negation.
theorem real_neg_pos(x: Real) {
    x < Real.0 implies -x > Real.0
} by {
    if x < Real.0 {
        lt_imp_lte(x, Real.0)
        x <= Real.0
        lte_add_right(x, Real.0, -x)
        x + -x <= Real.0 + -x
        x + -x = Real.0
        Real.0 <= Real.0 + -x
        Real.0 + -x = -x
        Real.0 <= -x
        -x >= Real.0
        if -x = Real.0 {
            x = Real.0
            false
        }
        -x != Real.0
        -x > Real.0
    }
}

/// A quotient of a positive by a positive product is positive.
theorem real_div_pos_three(a: Real, b: Real, c: Real) {
    a > Real.0 and b > Real.0 and c > Real.0 implies a / (b * c) > Real.0
} by {
    if a > Real.0 and b > Real.0 and c > Real.0 {
        mul_pos_pos(b, c)
        b.is_positive and c.is_positive implies (b * c).is_positive
        b.is_positive
        c.is_positive
        (b * c).is_positive
        b * c > Real.0
        real_div_pos(a, b * c)
        a > Real.0 and b * c > Real.0 implies a / (b * c) > Real.0
        a / (b * c) > Real.0
    }
}

/// The tail contribution is bounded by a quarter of `rho / a`.
theorem t_mul_h_bound(rho: Real, aa: Real, hh: Real, t: Real) {
    rho > Real.0 and aa > Real.0 and hh >= Real.0 and t >= Real.0 and t <= rho / (real_four * aa * (hh + Real.1))
    implies t * hh <= rho / (real_four * aa)
} by {
    if rho > Real.0 and aa > Real.0 and hh >= Real.0 and t >= Real.0 and t <= rho / (real_four * aa * (hh + Real.1)) {
        mul_le_mul_of_nonneg_right(t, rho / (real_four * aa * (hh + Real.1)), hh)
        t <= rho / (real_four * aa * (hh + Real.1)) and hh >= Real.0 implies t * hh <= rho / (real_four * aa * (hh + Real.1)) * hh
        t * hh <= rho / (real_four * aa * (hh + Real.1)) * hh
        real_plus_one_pos(hh)
        hh + Real.1 > Real.0
        real_four_pos
        real_four > Real.0
        real_four.is_positive
        aa > Real.0
        aa.is_positive
        mul_pos_pos(real_four, aa)
        real_four.is_positive and aa.is_positive implies (real_four * aa).is_positive
        (real_four * aa).is_positive
        real_four * aa > Real.0
        real_mul_pos_ne_zero(real_four * aa, hh + Real.1)
        real_four * aa > Real.0 and hh + Real.1 > Real.0
        real_four * aa * (hh + Real.1) != Real.0
        real_four * aa != Real.0
        hh + Real.1 != Real.0
        real_inverse_mul(real_four * aa, hh + Real.1)
        ((real_four * aa) * (hh + Real.1)).inverse = (real_four * aa).inverse * (hh + Real.1).inverse
        rho / (real_four * aa * (hh + Real.1)) = rho * ((real_four * aa) * (hh + Real.1)).inverse
        rho * ((real_four * aa) * (hh + Real.1)).inverse = rho * ((real_four * aa).inverse * (hh + Real.1).inverse)
        rho * ((real_four * aa).inverse * (hh + Real.1).inverse) = (rho * (real_four * aa).inverse) * (hh + Real.1).inverse
        rho / (real_four * aa * (hh + Real.1)) * hh = (rho * (real_four * aa).inverse) * ((hh + Real.1).inverse * hh)
        rho * (real_four * aa).inverse = rho / (real_four * aa)
        (hh + Real.1).inverse * hh = hh * (hh + Real.1).inverse
        rho / (real_four * aa * (hh + Real.1)) * hh = (rho / (real_four * aa)) * (hh * (hh + Real.1).inverse)
        hh * (hh + Real.1).inverse = hh / (hh + Real.1)
        rho / (real_four * aa * (hh + Real.1)) * hh = (rho / (real_four * aa)) * (hh / (hh + Real.1))
        lte_self(hh)
        hh <= hh
        real_one_positive
        Real.1 > Real.0
        lt_imp_lte(Real.0, Real.1)
        Real.0 <= Real.1
        add_lte_add(hh, hh, Real.0, Real.1)
        hh <= hh and Real.0 <= Real.1 implies hh + Real.0 <= hh + Real.1
        hh + Real.0 <= hh + Real.1
        hh + Real.0 = hh
        hh <= hh + Real.1
        (hh + Real.1).inverse > Real.0
        lte_add_right(hh, hh + Real.1, (hh + Real.1).inverse)
        mul_le_mul_of_nonneg_right(hh, hh + Real.1, (hh + Real.1).inverse)
        hh <= hh + Real.1 and (hh + Real.1).inverse >= Real.0 implies hh * (hh + Real.1).inverse <= (hh + Real.1) * (hh + Real.1).inverse
        (hh + Real.1).inverse >= Real.0
        hh * (hh + Real.1).inverse <= (hh + Real.1) * (hh + Real.1).inverse
        (hh + Real.1) * (hh + Real.1).inverse = Real.1
        hh * (hh + Real.1).inverse <= Real.1
        hh / (hh + Real.1) = hh * (hh + Real.1).inverse
        hh / (hh + Real.1) <= Real.1
        real_div_pos(rho, real_four * aa)
        rho > Real.0 and real_four * aa > Real.0 implies rho / (real_four * aa) > Real.0
        rho > Real.0
        rho / (real_four * aa) > Real.0
        rho / (real_four * aa) >= Real.0
        mul_le_mul_of_nonneg_right(hh / (hh + Real.1), Real.1, rho / (real_four * aa))
        hh / (hh + Real.1) <= Real.1 and rho / (real_four * aa) >= Real.0 implies hh / (hh + Real.1) * (rho / (real_four * aa)) <= Real.1 * (rho / (real_four * aa))
        hh / (hh + Real.1) * (rho / (real_four * aa)) <= Real.1 * (rho / (real_four * aa))
        real_mul_comm(hh / (hh + Real.1), rho / (real_four * aa))
        (rho / (real_four * aa)) * (hh / (hh + Real.1)) <= Real.1 * (rho / (real_four * aa))
        Real.1 * (rho / (real_four * aa)) = rho / (real_four * aa)
        (rho / (real_four * aa)) * (hh / (hh + Real.1)) <= rho / (real_four * aa)
        rho / (real_four * aa * (hh + Real.1)) * hh <= rho / (real_four * aa)
        lte_trans(t * hh, rho / (real_four * aa * (hh + Real.1)) * hh, rho / (real_four * aa))
        t * hh <= rho / (real_four * aa)
    }
}

/// A positive base raised to a power stays positive.
theorem real_pow_pos(x: Real, n: Nat) {
    x > Real.0 implies x.pow(n) > Real.0
} by {
    define p(k: Nat) -> Bool {
        x > Real.0 implies x.pow(k) > Real.0
    }
    pow_zero(x)
    x.pow(Nat.0) = Real.1
    real_one_positive
    Real.1 > Real.0
    p(Nat.0)
    forall(k: Nat) {
        if p(k) {
            if x > Real.0 {
                x.pow(k) > Real.0
                real_pow_suc(x, k)
                x.pow(k.suc) = x * x.pow(k)
                x.is_positive
                x.pow(k).is_positive
                mul_pos_pos(x, x.pow(k))
                x.is_positive and x.pow(k).is_positive implies (x * x.pow(k)).is_positive
                (x * x.pow(k)).is_positive
                x * x.pow(k) > Real.0
                x.pow(k.suc) > Real.0
            }
            p(k.suc)
        }
    }
    p(Nat.0) and forall(k: Nat) {
        p(k) implies p(k.suc)
    }
    alt_induction(p)
    p(n)
}

/// `a - (x - y) = a + (-x + y)`.
theorem real_sub_sub_expand(a: Real, x: Real, y: Real) {
    a - (x - y) = a + (-x + y)
} by {
    x - y = x + -y
    a - (x - y) = a + -(x - y)
    a + -(x - y) = a + -(x + -y)
    neg_distrib(x, -y)
    -(x + -y) = -x + -(-y)
    neg_neg(y)
    -(-y) = y
    -(x + -y) = -x + y
    a - (x - y) = a + (-x + y)
}

/// `two * r - x = r + (r - x)`.
theorem real_two_sub_split(r: Real, x: Real) {
    two * r - x = r + (r - x)
} by {
    two_mul_eq_add_self_real(r)
    two * r = r + r
    two * r - x = (r + r) - x
    (r + r) - x = r + (r - x)
    two * r - x = r + (r - x)
}

/// Subtracting a nonnegative amount does not increase a number.
theorem real_sub_le_self(x: Real, y: Real) {
    y >= Real.0 implies x - y <= x
} by {
    if y >= Real.0 {
        lte_add_right(Real.0, y, -y)
        Real.0 <= y implies Real.0 + -y <= y + -y
        Real.0 + -y <= y + -y
        Real.0 + -y = -y
        y + -y = Real.0
        y + -y = Real.0 implies Real.0 + -y <= Real.0
        Real.0 + -y <= Real.0
        -y <= Real.0
        lte_add_right(-y, Real.0, x)
        -y <= Real.0 implies -y + x <= Real.0 + x
        -y + x <= x
        x + -y <= x
        x - y = x + -y
        x - y <= x
    }
}

/// Dividing both sides of a non-strict inequality by a positive number.
theorem real_div_le_div(a: Real, b: Real, c: Real) {
    a <= b and c > Real.0 implies a / c <= b / c
} by {
    if a <= b and c > Real.0 {
        c != Real.0
        real_inv_pos(c)
        c.inverse > Real.0
        c.inverse >= Real.0
        mul_le_mul_of_nonneg_right(a, b, c.inverse)
        a <= b and c.inverse >= Real.0 implies a * c.inverse <= b * c.inverse
        a * c.inverse <= b * c.inverse
        a / c = a * c.inverse
        b / c = b * c.inverse
        a / c <= b / c
    }
}

/// `a - s * u + s * v = a - s * (u - v)`.
theorem real_sub_combine(a: Real, s: Real, u: Real, v: Real) {
    a - s * u + s * v = a - s * (u - v)
} by {
    a - s * u + s * v = a + (-(s * u) + s * v)
    mul_sub_left(s, u, v)
    s * (u - v) = s * u - s * v
    s * u - s * v = s * u + -(s * v)
    -(s * (u - v)) = -(s * u + -(s * v))
    neg_distrib(s * u, -(s * v))
    -(s * u + -(s * v)) = -(s * u) + -(-(s * v))
    neg_neg(s * v)
    -(-(s * v)) = s * v
    -(s * u + -(s * v)) = -(s * u) + s * v
    -(s * (u - v)) = -(s * u) + s * v
    a - s * (u - v) = a + (-(s * u) + s * v)
    a - s * u + s * v = a - s * (u - v)
}

/// Halving the denominator twice: `rho / (2a) = 2 * (rho / (4a))`.
theorem real_div_halve_twice(rho: Real, aa: Real) {
    rho > Real.0 and aa > Real.0 implies rho / (two * aa) = two * (rho / (real_four * aa))
} by {
    if rho > Real.0 and aa > Real.0 {
        real_four = two + two
        two_mul_eq_add_self_real(two)
        two * two = two + two
        real_four * aa = (two * two) * aa
        (two * two) * aa = (two * aa) * two
        real_four * aa = (two * aa) * two
        rho / (real_four * aa) = rho * (real_four * aa).inverse
        real_inverse_mul(two * aa, two)
        ((two * aa) * two).inverse = (two * aa).inverse * two.inverse
        (real_four * aa).inverse = (two * aa).inverse * two.inverse
        two * aa != Real.0
        real_two_positive
        two > Real.0
        real_mul_pos_ne_zero(two, aa)
        aa > Real.0
        two * aa != Real.0
        two != Real.0
        (real_four * aa).inverse = (two * aa).inverse * two.inverse
        rho * (real_four * aa).inverse = rho * ((two * aa).inverse * two.inverse)
        rho * ((two * aa).inverse * two.inverse) = (rho * (two * aa).inverse) * two.inverse
        two * (rho / (real_four * aa)) = two * ((rho * (two * aa).inverse) * two.inverse)
        two * ((rho * (two * aa).inverse) * two.inverse) = (rho * (two * aa).inverse) * (two * two.inverse)
        two * two.inverse = Real.1
        (rho * (two * aa).inverse) * (two * two.inverse) = (rho * (two * aa).inverse) * Real.1
        (rho * (two * aa).inverse) * Real.1 = rho * (two * aa).inverse
        two * (rho / (real_four * aa)) = rho * (two * aa).inverse
        rho * (two * aa).inverse = rho / (two * aa)
        rho / (two * aa) = two * (rho / (real_four * aa))
    }
}

/// The gap between `rho / (2a)` and a small tail is at least `rho / (4a)`.
theorem rho_half_gap(rho: Real, aa: Real, hh: Real, t: Real) {
    rho > Real.0 and aa > Real.0 and hh >= Real.0 and t >= Real.0 and t * hh <= rho / (real_four * aa)
    implies rho / (two * aa) - t * hh >= rho / (real_four * aa)
} by {
    if rho > Real.0 and aa > Real.0 and hh >= Real.0 and t >= Real.0 and t * hh <= rho / (real_four * aa) {
        real_div_halve_twice(rho, aa)
        rho / (two * aa) = two * (rho / (real_four * aa))
        real_sub_lte_lte(two * (rho / (real_four * aa)), t * hh, rho / (real_four * aa))
        t * hh <= rho / (real_four * aa) implies two * (rho / (real_four * aa)) - rho / (real_four * aa) <= two * (rho / (real_four * aa)) - t * hh
        two * (rho / (real_four * aa)) - rho / (real_four * aa) <= two * (rho / (real_four * aa)) - t * hh
        two * (rho / (real_four * aa)) - rho / (real_four * aa) = rho / (real_four * aa)
        rho / (real_four * aa) <= two * (rho / (real_four * aa)) - t * hh
        rho / (two * aa) - t * hh >= rho / (real_four * aa)
    }
}

/// Subtracting a positive amount strictly decreases a number.
theorem real_sub_lt_self(a: Real, x: Real) {
    x > Real.0 implies a - x < a
} by {
    if x > Real.0 {
        lt_imp_lte(Real.0, x)
        Real.0 <= x
        lte_add_right(Real.0, x, -x)
        Real.0 <= x implies Real.0 + -x <= x + -x
        Real.0 + -x = -x
        x + -x = Real.0
        x + -x = Real.0 implies Real.0 + -x <= Real.0
        Real.0 + -x <= Real.0
        -x <= Real.0
        if -x = Real.0 {
            x = Real.0
            false
        }
        -x != Real.0
        -x < Real.0
        lte_add_right(-x, Real.0, a)
        -x <= Real.0 implies -x + a <= Real.0 + a
        -x + a <= a
        a + -x <= a
        a - x = a + -x
        a - x <= a
        if a - x = a {
            a - x = a + -x
            a + -x = a
            a + -x = a + Real.0
            right_cancel(-x, Real.0, a)
            -x + a = Real.0 + a implies -x = Real.0
            -x + a = a + -x
            Real.0 + a = a + Real.0
            a + Real.0 = a
            -x = Real.0
            x = Real.0
            false
        }
        a - x != a
        a - x < a
    }
}

/// A number at least zero is zero at most it: the flip between `>=` and `<=`.
theorem real_gte_lte_flip(x: Real) {
    x >= Real.0 implies Real.0 <= x
} by {
}

/// `x / (x + 1) < 1` for `x >= 0`.
theorem real_div_lt_one(x: Real) {
    x >= Real.0 implies x / (x + Real.1) < Real.1
} by {
    if x >= Real.0 {
        lte_self(x)
        x <= x
        lte_add_right(x, x, Real.1)
        x <= x implies x + Real.1 <= x + Real.1
        x + Real.1 <= x + Real.1
        x < x + Real.1
        x + Real.1 > Real.0
        real_plus_one_pos(x)
        x + Real.1 > Real.0
        x + Real.1 != Real.0
        real_inv_pos(x + Real.1)
        (x + Real.1).inverse > Real.0
        mul_lt_mul_of_pos_right(x, x + Real.1, (x + Real.1).inverse)
        x < x + Real.1 and (x + Real.1).inverse > Real.0 implies x * (x + Real.1).inverse < (x + Real.1) * (x + Real.1).inverse
        x < x + Real.1
        (x + Real.1).inverse > Real.0
        x * (x + Real.1).inverse < (x + Real.1) * (x + Real.1).inverse
        (x + Real.1) * (x + Real.1).inverse = Real.1
        x * (x + Real.1).inverse < Real.1
        x / (x + Real.1) = x * (x + Real.1).inverse
        x / (x + Real.1) < Real.1
    }
}

/// `(a / b) * c = (a * c) / b`.
theorem real_div_mul_comm(a: Real, b: Real, c: Real) {
    (a / b) * c = (a * c) / b
} by {
    a / b = a * b.inverse
    (a / b) * c = (a * b.inverse) * c
    (a * b.inverse) * c = a * (b.inverse * c)
    b.inverse * c = c * b.inverse
    a * (b.inverse * c) = a * (c * b.inverse)
    a * (c * b.inverse) = (a * c) * b.inverse
    (a * c) / b = (a * c) * b.inverse
    (a / b) * c = (a * c) / b
}

/// `eps * bb / (bb + 1) < eps` for positive `eps` and nonnegative `bb`.
theorem eps_mul_div_lt_eps(eps: Real, bb: Real) {
    eps > Real.0 and bb >= Real.0 implies eps * bb / (bb + Real.1) < eps
} by {
    if eps > Real.0 and bb >= Real.0 {
        eps * bb / (bb + Real.1) = eps * (bb / (bb + Real.1))
        real_div_lt_one(bb)
        bb >= Real.0 implies bb / (bb + Real.1) < Real.1
        bb / (bb + Real.1) < Real.1
        mul_lt_mul_of_pos_right(bb / (bb + Real.1), Real.1, eps)
        bb / (bb + Real.1) < Real.1 and eps > Real.0 implies bb / (bb + Real.1) * eps < Real.1 * eps
        eps > Real.0
        bb / (bb + Real.1) * eps < Real.1 * eps
        bb / (bb + Real.1) * eps = eps * (bb / (bb + Real.1))
        eps * (bb / (bb + Real.1)) < Real.1 * eps
        Real.1 * eps = eps
        eps * (bb / (bb + Real.1)) < eps
        eps * bb / (bb + Real.1) < eps
    }
}

/// A nonzero natural number is at least one.
theorem nat_ne_zero_imp_ge_one(n: Nat) {
    n != Nat.0 implies n >= Nat.1
} by {
    if n != Nat.0 {
        zero_or_suc(n)
        if n = Nat.0 {
            false
        }
        let m: Nat satisfy {
            n = m.suc
        }
        n = m.suc
        nat_suc_ge_one(m)
        n >= Nat.1
    }
}

/// D'Alembert's lemma: if `p(z0) != 0` and `p` is nonconstant, some nearby
/// point evaluates to strictly smaller modulus.
theorem fta_perturbation(p: Polynomial[Complex], z0: Complex) {
    polynomial_eval(p, z0) != Complex.0 and
    exists(x: Complex, y: Complex) { polynomial_eval(p, x) != polynomial_eval(p, y) }
    implies exists(z1: Complex) {
        polynomial_eval(p, z1).modulus < polynomial_eval(p, z0).modulus
    }
} by {
    if polynomial_eval(p, z0) != Complex.0 and
        exists(x: Complex, y: Complex) { polynomial_eval(p, x) != polynomial_eval(p, y) } {
        let av: Complex = polynomial_eval(p, z0)
        let am: Real = av.modulus
        modulus_pos(av)
        av != Complex.0 implies av.modulus > Real.0
        am > Real.0
        am >= Real.0
        polynomial_support_bound_exists(p)
        let n: Nat satisfy {
            polynomial_support_bounded_by(p, n)
        }
        if n = Nat.0 {
            polynomial_zero_of_support_bound_zero(p)
            p = Polynomial[Complex].zero
            polynomial_eval_zero(z0)
            polynomial_eval(Polynomial[Complex].zero, z0) = Complex.0
            av = polynomial_eval(p, z0)
            av = Complex.0
            false
        }
        n != Nat.0
        nat_ne_zero_imp_ge_one(n)
        n >= Nat.1
        perturbation_first_coeff(p, z0, n)
        polynomial_support_bounded_by(p, n) and n >= Nat.1 and polynomial_eval(p, z0) != Complex.0 and
            exists(x: Complex, y: Complex) { polynomial_eval(p, x) != polynomial_eval(p, y) }
        exists(k: Nat) {
            k >= Nat.1 and k <= n - 1 and
            polynomial_eval(iterated_quotient(p, z0, n, k), z0) != Complex.0 and
            (forall(j: Nat) { j >= Nat.1 and j < k implies polynomial_eval(iterated_quotient(p, z0, n, j), z0) = Complex.0 })
        }
        let k: Nat satisfy {
            k >= Nat.1 and k <= n - 1 and
            polynomial_eval(iterated_quotient(p, z0, n, k), z0) != Complex.0 and
            (forall(j: Nat) { j >= Nat.1 and j < k implies polynomial_eval(iterated_quotient(p, z0, n, j), z0) = Complex.0 })
        }
        k >= Nat.1
        k <= n - 1
        polynomial_eval(iterated_quotient(p, z0, n, k), z0) != Complex.0
        forall(j: Nat) { j >= Nat.1 and j < k implies polynomial_eval(iterated_quotient(p, z0, n, j), z0) = Complex.0 }
        let ck: Complex = polynomial_eval(iterated_quotient(p, z0, n, k), z0)
        ck != Complex.0
        let b: Real = ck.modulus
        modulus_pos(ck)
        ck != Complex.0 implies ck.modulus > Real.0
        b > Real.0
        b >= Real.0
        let w0: Complex = omega(4 * k).pow(k)
        omega_four_k_sq(k)
        w0 * w0 = -Complex.1
        conj_ne_zero(av)
        av.conj != Complex.0
        let dd: Complex = av.conj * ck
        field_mul_nonzero(av.conj, ck)
        av.conj != Complex.0
        ck != Complex.0
        av.conj != Complex.0 and ck != Complex.0 implies av.conj * ck != Complex.0
        dd != Complex.0
        exists_neg_real_part_four(dd, w0)
        dd != Complex.0 and w0 * w0 = -Complex.1
        exists(j: Nat) {
            j <= 3 and (dd * w0.pow(j)).re < Real.0
        }
        let j0: Nat satisfy {
            j0 <= 3 and (dd * w0.pow(j0)).re < Real.0
        }
        (dd * w0.pow(j0)).re < Real.0
        let rho: Real = -(dd * w0.pow(j0)).re
        rho > Real.0
        rho >= Real.0
        let u: Complex = omega(4 * k).pow(j0)
        omega_four_k_unit_modulus(k)
        omega(4 * k).modulus = Real.1
        modulus_unit_pow(omega(4 * k), j0)
        u.modulus = Real.1
        complex_pow_pow(omega(4 * k), j0, k)
        (omega(4 * k).pow(j0)).pow(k) = omega(4 * k).pow(j0 * k)
        complex_pow_pow(omega(4 * k), k, j0)
        (omega(4 * k).pow(k)).pow(j0) = omega(4 * k).pow(k * j0)
        j0 * k = k * j0
        u.pow(k) = w0.pow(j0)
        mul_comm(av.conj, ck)
        dd = av.conj * ck
        av.conj * ck = ck * av.conj
        dd = ck * av.conj
        dd * u.pow(k) = ck * av.conj * u.pow(k)
        mul_assoc(ck, av.conj, u.pow(k))
        ck * av.conj * u.pow(k) = ck * (av.conj * u.pow(k))
        av.conj * u.pow(k) = u.pow(k) * av.conj
        mul_comm(av.conj, u.pow(k))
        ck * (av.conj * u.pow(k)) = ck * (u.pow(k) * av.conj)
        mul_assoc(ck, u.pow(k), av.conj)
        ck * (u.pow(k) * av.conj) = (ck * u.pow(k)) * av.conj
        dd * u.pow(k) = (ck * u.pow(k)) * av.conj
        re_mul_conj_pow(av, ck, u, k)
        (av * (ck * u.pow(k)).conj).re = (av.conj * ck * u.pow(k)).re
        av.conj * ck * u.pow(k) = (av.conj * ck) * u.pow(k)
        mul_assoc(av.conj, ck, u.pow(k))
        (av.conj * ck * u.pow(k)).re = (dd * u.pow(k)).re
        (av * (ck * u.pow(k)).conj).re = (dd * u.pow(k)).re
        (dd * u.pow(k)).re = (dd * w0.pow(j0)).re
        u.pow(k) = w0.pow(j0)
        (dd * u.pow(k)).re = (dd * w0.pow(j0)).re
        (av * (ck * u.pow(k)).conj).re = (dd * w0.pow(j0)).re
        (dd * w0.pow(j0)).re < Real.0
        (av * (ck * u.pow(k)).conj).re < Real.0
        (av * (ck * u.pow(k)).conj).re = -rho
        rho = -(dd * w0.pow(j0)).re
        (av * (ck * u.pow(k)).conj).re = -rho
        rho = -(av * (ck * u.pow(k)).conj).re
        let qs: Polynomial[Complex] = iterated_quotient(p, z0, n, k.suc)
        lte_suc_suc(k, n - 1)
        k <= n - 1
        k.suc <= n - 1 + 1
        nat_suc_pred_self(n)
        (n - 1).suc = n
        n - 1 + 1 = n
        k.suc <= n
        iterated_quotient_support_bounded(p, z0, n, k.suc)
        polynomial_support_bounded_by(p, n) and k.suc <= n
        polynomial_support_bounded_by(qs, n - k.suc)
        nat_pred_lt_self(n)
        n - 1 < n
        lte_lt_trans(k, n - 1, n)
        k < n
        nat_sub_sub_suc(k, n)
        n - k - 1 = n - k.suc
        n - k.suc = n - k - 1
        polynomial_support_bounded_by(qs, n - k - 1)
        let hh: Real = range_sum(coeff_pow_bound_term(qs.coeff, z0.modulus + Real.1), n - k - 1)
        forall(i: Nat) {
            if i < n - k - 1 {
                coeff_pow_bound_term(qs.coeff, z0.modulus + Real.1, i) = qs.coeff(i).modulus * (z0.modulus + Real.1).pow(i)
                modulus_nonneg(qs.coeff(i))
                qs.coeff(i).modulus >= Real.0
                modulus_nonneg(z0)
                z0.modulus >= Real.0
                real_one_positive
                Real.1 > Real.0
                lt_imp_lte(Real.0, Real.1)
                Real.0 <= Real.1
                add_lte_add(z0.modulus, z0.modulus, Real.0, Real.1)
                z0.modulus <= z0.modulus and Real.0 <= Real.1 implies z0.modulus + Real.0 <= z0.modulus + Real.1
                z0.modulus <= z0.modulus
                z0.modulus + Real.0 <= z0.modulus + Real.1
                z0.modulus + Real.0 = z0.modulus
                z0.modulus <= z0.modulus + Real.1
                z0.modulus + Real.1 >= Real.0
                real_pow_nonneg(z0.modulus + Real.1, i)
                (z0.modulus + Real.1).pow(i) >= Real.0
                mul_nonneg(qs.coeff(i).modulus, (z0.modulus + Real.1).pow(i))
                qs.coeff(i).modulus >= Real.0 and (z0.modulus + Real.1).pow(i) >= Real.0 implies qs.coeff(i).modulus * (z0.modulus + Real.1).pow(i) >= Real.0
                coeff_pow_bound_term(qs.coeff, z0.modulus + Real.1, i) >= Real.0
            }
        }
        forall(i: Nat) { i < n - k - 1 implies coeff_pow_bound_term(qs.coeff, z0.modulus + Real.1, i) >= Real.0 }
        range_sum_nonneg_real(coeff_pow_bound_term(qs.coeff, z0.modulus + Real.1), n - k - 1)
        hh >= Real.0
        perturbation_t_choice(rho, b, am, hh)
        rho > Real.0 and b > Real.0 and am > Real.0 and hh >= Real.0
        exists(t: Real) {
            t > Real.0 and t <= Real.1 and t <= rho / (b * b) and t <= am * am / (real_four * rho) and t <= rho / (real_four * am * (hh + Real.1))
        }
        let t: Real satisfy {
            t > Real.0 and t <= Real.1 and t <= rho / (b * b) and t <= am * am / (real_four * rho) and t <= rho / (real_four * am * (hh + Real.1))
        }
        t > Real.0
        t <= Real.1
        t <= rho / (b * b)
        t <= am * am / (real_four * rho)
        t <= rho / (real_four * am * (hh + Real.1))
        t >= Real.0
        t_mul_h_bound(rho, am, hh, t)
        t * hh <= rho / (real_four * am)
        let w: Complex = Complex.from_real(t) * u
        let z1: Complex = z0 + w
        perturbation_expansion(p, z0, w, n, k)
        polynomial_support_bounded_by(p, n) and n >= Nat.1 and k >= Nat.1 and k <= n - 1 and
            polynomial_eval(iterated_quotient(p, z0, n, k), z0) != Complex.0 and
            (forall(j: Nat) { j >= Nat.1 and j < k implies polynomial_eval(iterated_quotient(p, z0, n, j), z0) = Complex.0 })
        polynomial_eval(p, z0 + w) = polynomial_eval(p, z0) +
            polynomial_eval(iterated_quotient(p, z0, n, k), z0) * w.pow(k) +
            w.pow(k.suc) * polynomial_eval(iterated_quotient(p, z0, n, k.suc), z0 + w)
        polynomial_eval(p, z1) = av + ck * w.pow(k) + w.pow(k.suc) * polynomial_eval(qs, z0 + w)
        modulus_triangle(av + ck * w.pow(k), w.pow(k.suc) * polynomial_eval(qs, z0 + w))
        (av + ck * w.pow(k) + w.pow(k.suc) * polynomial_eval(qs, z0 + w)).modulus <= (av + ck * w.pow(k)).modulus + (w.pow(k.suc) * polynomial_eval(qs, z0 + w)).modulus
        polynomial_eval(p, z1).modulus <= (av + ck * w.pow(k)).modulus + (w.pow(k.suc) * polynomial_eval(qs, z0 + w)).modulus
        modulus_mul(w.pow(k.suc), polynomial_eval(qs, z0 + w))
        (w.pow(k.suc) * polynomial_eval(qs, z0 + w)).modulus = w.pow(k.suc).modulus * polynomial_eval(qs, z0 + w).modulus
        w = Complex.from_real(t) * u
        modulus_mul(Complex.from_real(t), u)
        (Complex.from_real(t) * u).modulus = Complex.from_real(t).modulus * u.modulus
        modulus_from_real_nonneg(t)
        t >= Real.0 implies Complex.from_real(t).modulus = t
        Complex.from_real(t).modulus = t
        u.modulus = Real.1
        w.modulus = t * Real.1
        t * Real.1 = t
        w.modulus = t
        w.modulus <= Real.1
        lte_trans(t, Real.1, Real.1)
        lte_self(Real.1)
        Real.1 <= Real.1
        t <= Real.1
        w.modulus <= Real.1
        complex_modulus_pow(w, k.suc)
        w.pow(k.suc).modulus = w.modulus.pow(k.suc)
        real_pow_le_one(w.modulus, k.suc)
        w.modulus >= Real.0 and w.modulus <= Real.1 implies w.modulus.pow(k.suc) <= Real.1
        w.modulus >= Real.0
        w.pow(k.suc).modulus <= Real.1
        modulus_triangle(z0, w)
        (z0 + w).modulus <= z0.modulus + w.modulus
        w.modulus <= Real.1
        z0.modulus + w.modulus <= z0.modulus + Real.1
        add_lte_add(z0.modulus, z0.modulus, w.modulus, Real.1)
        w.modulus <= Real.1
        lte_trans((z0 + w).modulus, z0.modulus + w.modulus, z0.modulus + Real.1)
        (z0 + w).modulus <= z0.modulus + Real.1
        modulus_nonneg(z0)
        z0.modulus >= Real.0
        real_one_positive
        Real.1 > Real.0
        lt_imp_lte(Real.0, Real.1)
        Real.0 <= Real.1
        add_lte_add(z0.modulus, z0.modulus, Real.0, Real.1)
        z0.modulus <= z0.modulus and Real.0 <= Real.1 implies z0.modulus + Real.0 <= z0.modulus + Real.1
        z0.modulus <= z0.modulus
        z0.modulus + Real.0 <= z0.modulus + Real.1
        z0.modulus + Real.0 = z0.modulus
        z0.modulus <= z0.modulus + Real.1
        z0.modulus + Real.1 >= Real.0
        polynomial_eval_disk_bound(qs, z0 + w, z0.modulus + Real.1, n - k - 1)
        polynomial_support_bounded_by(qs, n - k - 1) and z0.modulus + Real.1 >= Real.0 and (z0 + w).modulus <= z0.modulus + Real.1
        polynomial_eval(qs, z0 + w).modulus <= range_sum(coeff_pow_bound_term(qs.coeff, z0.modulus + Real.1), n - k - 1)
        polynomial_eval(qs, z0 + w).modulus <= hh
        let sval: Real = t.pow(k)
        w.pow(k.suc).modulus >= Real.0
        mul_le_mul_of_nonneg_right(polynomial_eval(qs, z0 + w).modulus, hh, w.pow(k.suc).modulus)
        polynomial_eval(qs, z0 + w).modulus <= hh and w.pow(k.suc).modulus >= Real.0 implies polynomial_eval(qs, z0 + w).modulus * w.pow(k.suc).modulus <= hh * w.pow(k.suc).modulus
        polynomial_eval(qs, z0 + w).modulus * w.pow(k.suc).modulus <= hh * w.pow(k.suc).modulus
        (w.pow(k.suc) * polynomial_eval(qs, z0 + w)).modulus = w.pow(k.suc).modulus * polynomial_eval(qs, z0 + w).modulus
        (w.pow(k.suc) * polynomial_eval(qs, z0 + w)).modulus <= hh * w.pow(k.suc).modulus
        complex_modulus_pow(w, k.suc)
        w.pow(k.suc).modulus = w.modulus.pow(k.suc)
        w.modulus = t
        w.pow(k.suc).modulus = t.pow(k.suc)
        real_pow_suc(t, k)
        t.pow(k.suc) = t * t.pow(k)
        sval = t.pow(k)
        t.pow(k.suc) = t * sval
        w.pow(k.suc).modulus = t * sval
        hh * w.pow(k.suc).modulus = hh * (t * sval)
        hh * (t * sval) = t * sval * hh
        (w.pow(k.suc) * polynomial_eval(qs, z0 + w)).modulus <= t * sval * hh
        let sum_mod: Real = (av + ck * w.pow(k)).modulus
        abs_squared_add(av, ck * w.pow(k))
        (av + ck * w.pow(k)).abs_squared =
            av.abs_squared + (ck * w.pow(k)).abs_squared + ((av * (ck * w.pow(k)).conj).re + (av * (ck * w.pow(k)).conj).re)
        modulus_squared(av)
        av.modulus * av.modulus = av.abs_squared
        am * am = av.abs_squared
        (av + ck * w.pow(k)).abs_squared = am * am + (ck * w.pow(k)).abs_squared + ((av * (ck * w.pow(k)).conj).re + (av * (ck * w.pow(k)).conj).re)
        modulus_squared(ck * w.pow(k))
        (ck * w.pow(k)).modulus * (ck * w.pow(k)).modulus = (ck * w.pow(k)).abs_squared
        modulus_mul(ck, w.pow(k))
        (ck * w.pow(k)).modulus = ck.modulus * w.pow(k).modulus
        complex_modulus_pow(w, k)
        w.pow(k).modulus = w.modulus.pow(k)
        w.modulus = t
        w.pow(k).modulus = t.pow(k)
        (ck * w.pow(k)).modulus = b * t.pow(k)
        b * t.pow(k) = b * sval
        sval = t.pow(k)
        (ck * w.pow(k)).modulus = b * sval
        (ck * w.pow(k)).modulus * (ck * w.pow(k)).modulus = (b * sval) * (b * sval)
        (ck * w.pow(k)).abs_squared = (b * sval) * (b * sval)
        complex_pow_mul_distrib(Complex.from_real(t), u, k)
        (Complex.from_real(t) * u).pow(k) = Complex.from_real(t).pow(k) * u.pow(k)
        from_real_pow(t, k)
        Complex.from_real(t).pow(k) = Complex.from_real(t.pow(k))
        sval = t.pow(k)
        Complex.from_real(t).pow(k) = Complex.from_real(sval)
        w = Complex.from_real(t) * u
        w.pow(k) = Complex.from_real(sval) * u.pow(k)
        conj_mul(ck, w.pow(k))
        (ck * w.pow(k)).conj = ck.conj * w.pow(k).conj
        conj_mul(Complex.from_real(sval), u.pow(k))
        (Complex.from_real(sval) * u.pow(k)).conj = Complex.from_real(sval).conj * u.pow(k).conj
        conj_from_real(sval)
        Complex.from_real(sval).conj = Complex.from_real(sval)
        w.pow(k).conj = Complex.from_real(sval) * u.pow(k).conj
        (ck * w.pow(k)).conj = ck.conj * (Complex.from_real(sval) * u.pow(k).conj)
        av * (ck * w.pow(k)).conj = av * (ck.conj * (Complex.from_real(sval) * u.pow(k).conj))
        mul_assoc(ck.conj, Complex.from_real(sval), u.pow(k).conj)
        ck.conj * (Complex.from_real(sval) * u.pow(k).conj) = (ck.conj * Complex.from_real(sval)) * u.pow(k).conj
        mul_comm(ck.conj, Complex.from_real(sval))
        ck.conj * Complex.from_real(sval) = Complex.from_real(sval) * ck.conj
        (ck.conj * Complex.from_real(sval)) * u.pow(k).conj = (Complex.from_real(sval) * ck.conj) * u.pow(k).conj
        av * (ck.conj * (Complex.from_real(sval) * u.pow(k).conj)) = av * ((Complex.from_real(sval) * ck.conj) * u.pow(k).conj)
        mul_assoc(Complex.from_real(sval), ck.conj, u.pow(k).conj)
        (Complex.from_real(sval) * ck.conj) * u.pow(k).conj = Complex.from_real(sval) * (ck.conj * u.pow(k).conj)
        av * ((Complex.from_real(sval) * ck.conj) * u.pow(k).conj) = av * (Complex.from_real(sval) * (ck.conj * u.pow(k).conj))
        mul_assoc(av, Complex.from_real(sval), ck.conj * u.pow(k).conj)
        av * (Complex.from_real(sval) * (ck.conj * u.pow(k).conj)) = (av * Complex.from_real(sval)) * (ck.conj * u.pow(k).conj)
        mul_comm(av, Complex.from_real(sval))
        av * Complex.from_real(sval) = Complex.from_real(sval) * av
        (av * Complex.from_real(sval)) * (ck.conj * u.pow(k).conj) = (Complex.from_real(sval) * av) * (ck.conj * u.pow(k).conj)
        mul_assoc(Complex.from_real(sval), av, ck.conj * u.pow(k).conj)
        (Complex.from_real(sval) * av) * (ck.conj * u.pow(k).conj) = Complex.from_real(sval) * (av * (ck.conj * u.pow(k).conj))
        av * (ck.conj * (Complex.from_real(sval) * u.pow(k).conj)) = Complex.from_real(sval) * (av * (ck.conj * u.pow(k).conj))
        re_scalar_mul(sval, av * (ck.conj * u.pow(k).conj))
        (Complex.from_real(sval) * (av * (ck.conj * u.pow(k).conj))).re = sval * (av * (ck.conj * u.pow(k).conj)).re
        (av * (ck * w.pow(k)).conj).re = sval * (av * (ck.conj * u.pow(k).conj)).re
        conj_mul(ck, u.pow(k))
        (ck * u.pow(k)).conj = ck.conj * u.pow(k).conj
        av * (ck * u.pow(k)).conj = av * (ck.conj * u.pow(k).conj)
        (av * (ck * u.pow(k)).conj).re = -rho
        (av * (ck.conj * u.pow(k).conj)).re = -rho
        (av * (ck * w.pow(k)).conj).re = sval * -rho
        (av + ck * w.pow(k)).abs_squared = am * am + (b * sval) * (b * sval) + (sval * -rho + sval * -rho)
        two_mul_eq_add_self_real(sval * -rho)
        two * (sval * -rho) = sval * -rho + sval * -rho
        sval * -rho + sval * -rho = two * (sval * -rho)
        mul_neg_right(sval, rho)
        sval * -rho = -(sval * rho)
        two * (sval * -rho) = two * (-(sval * rho))
        mul_neg_right(two, sval * rho)
        two * (-(sval * rho)) = -(two * (sval * rho))
        two * (sval * -rho) = -two * (sval * rho)
        (av + ck * w.pow(k)).abs_squared = am * am + (b * sval) * (b * sval) + -two * (sval * rho)
        (av + ck * w.pow(k)).abs_squared = am * am + (b * sval) * (b * sval) - two * (sval * rho)
        -two * (sval * rho) = -two * rho * sval
        (av + ck * w.pow(k)).abs_squared = am * am + (b * sval) * (b * sval) - two * rho * sval
        modulus_squared(av + ck * w.pow(k))
        sum_mod * sum_mod = am * am + (b * sval) * (b * sval) - two * rho * sval
        am * am + (b * sval) * (b * sval) - two * rho * sval = am * am - two * rho * sval + (b * sval) * (b * sval)
        am * am - two * rho * sval + (b * sval) * (b * sval) = am * am + (-(two * rho * sval) + (b * sval) * (b * sval))
        two * rho * sval - (b * sval) * (b * sval) = two * rho * sval + -((b * sval) * (b * sval))
        neg_distrib(two * rho * sval, -((b * sval) * (b * sval)))
        -(two * rho * sval + -((b * sval) * (b * sval))) = -(two * rho * sval) + -(-((b * sval) * (b * sval)))
        neg_neg((b * sval) * (b * sval))
        -(-((b * sval) * (b * sval))) = (b * sval) * (b * sval)
        -(two * rho * sval - (b * sval) * (b * sval)) = -(two * rho * sval) + (b * sval) * (b * sval)
        am * am - (two * rho * sval - (b * sval) * (b * sval)) = am * am + (-(two * rho * sval) + (b * sval) * (b * sval))
        am * am - (two * rho * sval - (b * sval) * (b * sval)) = am * am - two * rho * sval + (b * sval) * (b * sval)
        am * am - two * rho * sval + (b * sval) * (b * sval) = am * am - (two * rho * sval - (b * sval) * (b * sval))
        sum_mod * sum_mod = am * am - (two * rho * sval - (b * sval) * (b * sval))
        am * am - (two * rho * sval - (b * sval) * (b * sval)) = am * am - two * rho * sval + (b * sval) * (b * sval)
        (b * sval) * (b * sval) = b * b * (sval * sval)
        am * am - two * rho * sval + (b * sval) * (b * sval) = am * am - two * rho * sval + b * b * sval * sval
        am * am - two * rho * sval + b * b * sval * sval = am * am - (two * rho * sval - b * b * sval * sval)
        sum_mod * sum_mod = am * am - (two * rho * sval - b * b * sval * sval)
        let xval: Real = two * rho * sval - b * b * sval * sval
        sum_mod * sum_mod = am * am - xval
        real_pow_le_self(t, k)
        sval <= t
        t <= rho / (b * b)
        lte_trans(sval, t, rho / (b * b))
        sval <= rho / (b * b)
        mul_le_mul_of_nonneg_right(sval, rho / (b * b), b * b)
        sval <= rho / (b * b) and b * b >= Real.0 implies sval * (b * b) <= rho / (b * b) * (b * b)
        b * b >= Real.0
        sval * (b * b) <= rho / (b * b) * (b * b)
        sval * (b * b) = b * b * sval
        real_mul_pos_ne_zero(b, b)
        b > Real.0
        b * b != Real.0
        mul_div_cancel(rho, b * b)
        rho / (b * b) * (b * b) = rho
        b * b * sval <= rho
        real_sub_lte_lte(rho, b * b * sval, rho)
        b * b * sval <= rho implies rho - rho <= rho - b * b * sval
        rho - rho = Real.0
        Real.0 <= rho - b * b * sval
        rho - b * b * sval >= Real.0
        two_mul_eq_add_self_real(rho)
        two * rho = rho + rho
        two * rho - b * b * sval = rho + (rho - b * b * sval)
        add_lte_add(rho, rho, Real.0, rho - b * b * sval)
        rho <= rho and Real.0 <= rho - b * b * sval implies rho + Real.0 <= rho + (rho - b * b * sval)
        lte_self(rho)
        rho <= rho
        rho + Real.0 <= rho + (rho - b * b * sval)
        rho + Real.0 = rho
        rho <= two * rho - b * b * sval
        two * rho - b * b * sval >= rho
        xval = two * rho * sval - b * b * sval * sval
        two * rho * sval - b * b * sval * sval = sval * (two * rho - b * b * sval)
        xval = sval * (two * rho - b * b * sval)
        mul_le_mul_of_nonneg_left(rho, two * rho - b * b * sval, sval)
        rho <= two * rho - b * b * sval and sval >= Real.0 implies sval * rho <= sval * (two * rho - b * b * sval)
        sval >= Real.0
        sval * rho <= sval * (two * rho - b * b * sval)
        xval = sval * (two * rho - b * b * sval)
        sval * (two * rho - b * b * sval) <= xval
        lte_trans(sval * rho, sval * (two * rho - b * b * sval), xval)
        xval >= rho * sval
        mul_nonneg(rho, sval)
        rho >= Real.0 and sval >= Real.0 implies rho * sval >= Real.0
        rho >= Real.0
        sval >= Real.0
        rho * sval >= Real.0
        real_gte_lte_flip(rho * sval)
        rho * sval >= Real.0 implies Real.0 <= rho * sval
        Real.0 <= rho * sval
        rho * sval <= xval
        lte_trans(Real.0, rho * sval, xval)
        Real.0 <= xval
        xval >= Real.0
        square_nonneg(b)
        b * b >= Real.0
        square_nonneg(sval)
        sval * sval >= Real.0
        mul_nonneg(b * b, sval * sval)
        b * b >= Real.0 and sval * sval >= Real.0 implies b * b * sval * sval >= Real.0
        b * b * sval * sval >= Real.0
        real_sub_lte_lte(two * rho * sval, Real.0, b * b * sval * sval)
        Real.0 <= b * b * sval * sval implies two * rho * sval - b * b * sval * sval <= two * rho * sval - Real.0
        two * rho * sval - Real.0 = two * rho * sval
        two * rho * sval - b * b * sval * sval <= two * rho * sval
        xval = two * rho * sval - b * b * sval * sval
        xval <= two * rho * sval
        mul_le_mul_of_nonneg_left(sval, t, two * rho)
        sval <= t and two * rho >= Real.0 implies two * rho * sval <= two * rho * t
        two * rho >= Real.0
        two * rho * sval <= two * rho * t
        lte_trans(xval, two * rho * sval, two * rho * t)
        xval <= two * rho * t
        t <= am * am / (real_four * rho)
        mul_le_mul_of_nonneg_left(t, am * am / (real_four * rho), two * rho)
        t <= am * am / (real_four * rho) and two * rho >= Real.0 implies two * rho * t <= two * rho * (am * am / (real_four * rho))
        real_four = two + two
        two_mul_eq_add_self_real(two)
        two * two = two + two
        two * rho * (am * am / (real_four * rho)) = (two * rho * am * am) / (real_four * rho)
        real_four * rho = (two * two) * rho
        (two * rho * am * am) / ((two * two) * rho) = (am * am * (two * rho)) / (two * (two * rho))
        div_cancel_common(am * am, two, two * rho)
        real_two_positive
        two > Real.0
        real_mul_pos_ne_zero(two, rho)
        two > Real.0 and rho > Real.0 implies two * rho != Real.0
        rho > Real.0
        two * rho != Real.0
        two != Real.0
        (am * am * (two * rho)) / (two * (two * rho)) = am * am / two
        two * rho * (am * am / (real_four * rho)) = am * am / two
        two * rho * t <= am * am / two
        lte_trans(xval, two * rho * t, am * am / two)
        xval <= am * am / two
        modulus_square_bound(av + ck * w.pow(k), am, xval)
        am > Real.0 and xval >= Real.0 and xval <= am * am / two and (am * am - xval) = (av + ck * w.pow(k)).modulus * (av + ck * w.pow(k)).modulus
        sum_mod <= am - xval / (two * am)
        real_two_positive
        two > Real.0
        am > Real.0
        two.is_positive
        am.is_positive
        mul_pos_pos(two, am)
        two.is_positive and am.is_positive implies (two * am).is_positive
        (two * am).is_positive
        two * am > Real.0
        real_div_le_div(rho * sval, xval, two * am)
        rho * sval <= xval and two * am > Real.0 implies rho * sval / (two * am) <= xval / (two * am)
        xval >= rho * sval
        rho * sval <= xval
        rho * sval / (two * am) <= xval / (two * am)
        xval / (two * am) >= rho * sval / (two * am)
        real_sub_lte_lte(am, rho * sval / (two * am), xval / (two * am))
        rho * sval / (two * am) <= xval / (two * am) implies am - xval / (two * am) <= am - rho * sval / (two * am)
        am - xval / (two * am) <= am - rho * sval / (two * am)
        lte_trans(sum_mod, am - xval / (two * am), am - rho * sval / (two * am))
        sum_mod <= am - rho * sval / (two * am)
        (w.pow(k.suc) * polynomial_eval(qs, z0 + w)).modulus <= t * sval * hh
        lte_add_right((w.pow(k.suc) * polynomial_eval(qs, z0 + w)).modulus, t * sval * hh, sum_mod)
        (w.pow(k.suc) * polynomial_eval(qs, z0 + w)).modulus <= t * sval * hh implies (w.pow(k.suc) * polynomial_eval(qs, z0 + w)).modulus + sum_mod <= t * sval * hh + sum_mod
        (w.pow(k.suc) * polynomial_eval(qs, z0 + w)).modulus + sum_mod <= t * sval * hh + sum_mod
        polynomial_eval(p, z1).modulus <= sum_mod + (w.pow(k.suc) * polynomial_eval(qs, z0 + w)).modulus
        sum_mod + (w.pow(k.suc) * polynomial_eval(qs, z0 + w)).modulus = (w.pow(k.suc) * polynomial_eval(qs, z0 + w)).modulus + sum_mod
        lte_trans(polynomial_eval(p, z1).modulus, sum_mod + (w.pow(k.suc) * polynomial_eval(qs, z0 + w)).modulus, t * sval * hh + sum_mod)
        polynomial_eval(p, z1).modulus <= t * sval * hh + sum_mod
        t * sval * hh + sum_mod = sum_mod + t * sval * hh
        polynomial_eval(p, z1).modulus <= sum_mod + t * sval * hh
        add_lte_add(sum_mod, am - rho * sval / (two * am), t * sval * hh, t * sval * hh)
        sum_mod <= am - rho * sval / (two * am) and t * sval * hh <= t * sval * hh implies sum_mod + t * sval * hh <= am - rho * sval / (two * am) + t * sval * hh
        t * sval * hh <= t * sval * hh
        lte_self(t * sval * hh)
        t * sval * hh <= t * sval * hh
        lte_trans(polynomial_eval(p, z1).modulus, sum_mod + t * sval * hh, am - rho * sval / (two * am) + t * sval * hh)
        polynomial_eval(p, z1).modulus <= am - rho * sval / (two * am) + t * sval * hh
        real_sub_combine(am, sval, rho / (two * am), t * hh)
        am - sval * (rho / (two * am)) + sval * (t * hh) = am - sval * (rho / (two * am) - t * hh)
        am - rho * sval / (two * am) + t * sval * hh = am - sval * (rho / (two * am) - t * hh)
        rho_half_gap(rho, am, hh, t)
        rho > Real.0 and am > Real.0 and hh >= Real.0 and t >= Real.0 and t * hh <= rho / (real_four * am)
        rho / (two * am) - t * hh >= rho / (real_four * am)
        sval > Real.0
        t.pow(k) > Real.0
        real_pow_pos(t, k)
        t > Real.0
        real_four_pos
        real_four > Real.0
        real_four.is_positive
        am > Real.0
        am.is_positive
        mul_pos_pos(real_four, am)
        real_four.is_positive and am.is_positive implies (real_four * am).is_positive
        (real_four * am).is_positive
        real_four * am > Real.0
        real_div_pos(rho, real_four * am)
        rho > Real.0 and real_four * am > Real.0 implies rho / (real_four * am) > Real.0
        rho / (real_four * am) > Real.0
        sval > Real.0
        sval.is_positive
        (rho / (real_four * am)).is_positive
        mul_pos_pos(sval, rho / (real_four * am))
        sval.is_positive and (rho / (real_four * am)).is_positive implies (sval * (rho / (real_four * am))).is_positive
        (sval * (rho / (real_four * am))).is_positive
        sval * (rho / (real_four * am)) > Real.0
        real_sub_lt_self(am, sval * (rho / (real_four * am)))
        sval * (rho / (real_four * am)) > Real.0 implies am - sval * (rho / (real_four * am)) < am
        am - sval * (rho / (real_four * am)) < am
        mul_le_mul_of_nonneg_left(rho / (real_four * am), rho / (two * am) - t * hh, sval)
        rho / (real_four * am) <= rho / (two * am) - t * hh and sval >= Real.0 implies sval * (rho / (real_four * am)) <= sval * (rho / (two * am) - t * hh)
        rho / (real_four * am) <= rho / (two * am) - t * hh
        sval >= Real.0
        sval * (rho / (real_four * am)) <= sval * (rho / (two * am) - t * hh)
        real_sub_lte_lte(am, sval * (rho / (real_four * am)), sval * (rho / (two * am) - t * hh))
        sval * (rho / (real_four * am)) <= sval * (rho / (two * am) - t * hh) implies am - sval * (rho / (two * am) - t * hh) <= am - sval * (rho / (real_four * am))
        am - sval * (rho / (two * am) - t * hh) <= am - sval * (rho / (real_four * am))
        lte_lt_trans(am - sval * (rho / (two * am) - t * hh), am - sval * (rho / (real_four * am)), am)
        am - sval * (rho / (two * am) - t * hh) < am
        polynomial_eval(p, z1).modulus <= am - sval * (rho / (two * am) - t * hh)
        lte_lt_trans(polynomial_eval(p, z1).modulus, am - sval * (rho / (two * am) - t * hh), am)
        polynomial_eval(p, z1).modulus < am
        av = polynomial_eval(p, z0)
        am = av.modulus
        polynomial_eval(p, z1).modulus < polynomial_eval(p, z0).modulus
        exists(z2: Complex) {
            polynomial_eval(p, z2).modulus < polynomial_eval(p, z0).modulus
        }
    }
}

// ---------------------------------------------------------------------------
// Section 7.  Assembly of the fundamental theorem of algebra.
//
// The remaining analytic ingredient is the extreme value theorem on a
// closed disk: `|p|` is continuous and grows to infinity, so it attains a
// global minimum.  The library does not yet provide compactness of closed
// disks in the complex plane (there is no `TopologicalSpace` instance for
// `Complex`, no two-dimensional Heine-Borel, and no Bolzano-Weierstrass),
// so the theorem below is stated with the extreme value property as an
// explicit hypothesis.  The extreme value theorem on a closed disk is
// recorded in Section 8 with a proof sketch of what is missing.
// ---------------------------------------------------------------------------

/// A minimum of `|p|` on a disk around the origin is a global minimum,
/// provided `|p|` grows to at least `|p(0)|` outside the disk.
theorem global_minimum_from_disk(p: Polynomial[Complex], z0: Complex, r: Real) {
    r >= Real.0 and
    (forall(z: Complex) {
        z.modulus <= r implies polynomial_eval(p, z0).modulus <= polynomial_eval(p, z).modulus
    }) and
    (forall(z: Complex) {
        z.modulus >= r implies polynomial_eval(p, z).modulus >= polynomial_eval(p, Complex.0).modulus
    })
    implies forall(z: Complex) {
        polynomial_eval(p, z0).modulus <= polynomial_eval(p, z).modulus
    }
} by {
    if r >= Real.0 and
        (forall(z: Complex) {
            z.modulus <= r implies polynomial_eval(p, z0).modulus <= polynomial_eval(p, z).modulus
        }) and
        (forall(z: Complex) {
            z.modulus >= r implies polynomial_eval(p, z).modulus >= polynomial_eval(p, Complex.0).modulus
        }) {
        modulus_of_zero
        Complex.0.modulus = Real.0
        lte_trans(Real.0, Real.0, r)
        lte_self(Real.0)
        Real.0 <= Real.0
        Complex.0.modulus <= r
        polynomial_eval(p, z0).modulus <= polynomial_eval(p, Complex.0).modulus
        forall(z: Complex) {
            lte_or_gte(z.modulus, r)
            if z.modulus <= r {
                z.modulus <= r implies polynomial_eval(p, z0).modulus <= polynomial_eval(p, z).modulus
                polynomial_eval(p, z0).modulus <= polynomial_eval(p, z).modulus
            } else {
                not z.modulus <= r
                r <= z.modulus
                z.modulus >= r
                z.modulus >= r implies polynomial_eval(p, z).modulus >= polynomial_eval(p, Complex.0).modulus
                polynomial_eval(p, z).modulus >= polynomial_eval(p, Complex.0).modulus
                lte_trans(polynomial_eval(p, z0).modulus, polynomial_eval(p, Complex.0).modulus, polynomial_eval(p, z).modulus)
                polynomial_eval(p, z0).modulus <= polynomial_eval(p, z).modulus
            }
        }
        forall(z: Complex) {
            polynomial_eval(p, z0).modulus <= polynomial_eval(p, z).modulus
        }
    }
}

/// The fundamental theorem of algebra, assuming the extreme value theorem:
/// if `|p|` attains a global minimum and `p` is nonconstant, that minimum
/// is a root of `p`.
theorem fundamental_theorem_of_algebra_of_extreme(p: Polynomial[Complex]) {
    exists(x: Complex, y: Complex) { polynomial_eval(p, x) != polynomial_eval(p, y) } and
    exists(z0: Complex) {
        forall(z: Complex) { polynomial_eval(p, z0).modulus <= polynomial_eval(p, z).modulus }
    }
    implies exists(z: Complex) { polynomial_eval(p, z) = Complex.0 }
} by {
    if exists(x: Complex, y: Complex) { polynomial_eval(p, x) != polynomial_eval(p, y) } and
        exists(z0: Complex) {
            forall(z: Complex) { polynomial_eval(p, z0).modulus <= polynomial_eval(p, z).modulus }
        } {
        let z0: Complex satisfy {
            forall(z: Complex) { polynomial_eval(p, z0).modulus <= polynomial_eval(p, z).modulus }
        }
        forall(z: Complex) { polynomial_eval(p, z0).modulus <= polynomial_eval(p, z).modulus }
        if polynomial_eval(p, z0) != Complex.0 {
            fta_perturbation(p, z0)
            polynomial_eval(p, z0) != Complex.0 and
                exists(x: Complex, y: Complex) { polynomial_eval(p, x) != polynomial_eval(p, y) }
            exists(z1: Complex) {
                polynomial_eval(p, z1).modulus < polynomial_eval(p, z0).modulus
            }
            let z1: Complex satisfy {
                polynomial_eval(p, z1).modulus < polynomial_eval(p, z0).modulus
            }
            polynomial_eval(p, z1).modulus < polynomial_eval(p, z0).modulus
            lt_imp_lte(polynomial_eval(p, z1).modulus, polynomial_eval(p, z0).modulus)
            polynomial_eval(p, z1).modulus <= polynomial_eval(p, z0).modulus
            forall(z: Complex) { polynomial_eval(p, z0).modulus <= polynomial_eval(p, z).modulus }
            polynomial_eval(p, z0).modulus <= polynomial_eval(p, z1).modulus
            lte_antisymm(polynomial_eval(p, z0).modulus, polynomial_eval(p, z1).modulus)
            polynomial_eval(p, z0).modulus = polynomial_eval(p, z1).modulus
            false
        }
        polynomial_eval(p, z0) = Complex.0
        exists(z: Complex) { polynomial_eval(p, z) = Complex.0 }
    }
}

// ---------------------------------------------------------------------------
// Section 8.  Continuity of polynomial evaluation.
// ---------------------------------------------------------------------------

/// A bound on the quotient of `p` by `X - z0` near `z0`: for
/// `|z - z0| <= 1`, `|q(z)|` is at most the disk bound at radius
/// `|z0| + 1`.
theorem quotient_local_bound(p: Polynomial[Complex], z0: Complex, z: Complex, n: Nat) {
    polynomial_support_bounded_by(p, n) and (z - z0).modulus <= Real.1
    implies polynomial_eval(iterated_quotient(p, z0, n, Nat.1), z).modulus
        <= range_sum(coeff_pow_bound_term(iterated_quotient(p, z0, n, Nat.1).coeff, z0.modulus + Real.1), n)
} by {
    if polynomial_support_bounded_by(p, n) and (z - z0).modulus <= Real.1 {
        polynomial_quotient_support_bounded_by(p, z0, n)
        polynomial_support_bounded_by(polynomial_quotient(p, z0, n), n)
        iterated_quotient_suc(p, z0, n, Nat.0)
        iterated_quotient(p, z0, n, Nat.1) = polynomial_quotient(iterated_quotient(p, z0, n, Nat.0), z0, n)
        iterated_quotient_zero(p, z0, n)
        iterated_quotient(p, z0, n, Nat.0) = p
        iterated_quotient(p, z0, n, Nat.1) = polynomial_quotient(p, z0, n)
        polynomial_support_bounded_by(iterated_quotient(p, z0, n, Nat.1), n)
        modulus_triangle(z0, z - z0)
        (z0 + (z - z0)).modulus <= z0.modulus + (z - z0).modulus
        (z0 + (z - z0)) = z
        z.modulus <= z0.modulus + (z - z0).modulus
        (z - z0).modulus <= Real.1
        z0.modulus + (z - z0).modulus <= z0.modulus + Real.1
        add_lte_add(z0.modulus, z0.modulus, (z - z0).modulus, Real.1)
        (z - z0).modulus <= Real.1
        lte_trans(z.modulus, z0.modulus + (z - z0).modulus, z0.modulus + Real.1)
        z.modulus <= z0.modulus + Real.1
        z0.modulus >= Real.0
        modulus_nonneg(z0)
        z0.modulus + Real.1 >= Real.0
        real_one_positive
        Real.1 > Real.0
        lt_imp_lte(Real.0, Real.1)
        Real.0 <= Real.1
        add_lte_add(z0.modulus, z0.modulus, Real.0, Real.1)
        z0.modulus <= z0.modulus and Real.0 <= Real.1 implies z0.modulus + Real.0 <= z0.modulus + Real.1
        z0.modulus <= z0.modulus
        z0.modulus + Real.0 <= z0.modulus + Real.1
        z0.modulus + Real.0 = z0.modulus
        z0.modulus <= z0.modulus + Real.1
        z0.modulus + Real.1 >= Real.0
        polynomial_eval_disk_bound(iterated_quotient(p, z0, n, Nat.1), z, z0.modulus + Real.1, n)
        polynomial_support_bounded_by(iterated_quotient(p, z0, n, Nat.1), n) and z0.modulus + Real.1 >= Real.0 and z.modulus <= z0.modulus + Real.1
        polynomial_eval(iterated_quotient(p, z0, n, Nat.1), z).modulus <= range_sum(coeff_pow_bound_term(iterated_quotient(p, z0, n, Nat.1).coeff, z0.modulus + Real.1), n)
    }
}

/// The epsilon-delta continuity condition at one epsilon.
theorem polynomial_eval_continuous_at_eps(p: Polynomial[Complex], z0: Complex, eps: Real) {
    eps > Real.0 implies exists(delta: Real) {
        delta > Real.0 and forall(z: Complex) {
            (z - z0).modulus < delta implies (polynomial_eval(p, z) - polynomial_eval(p, z0)).modulus < eps
        }
    }
} by {
    if eps > Real.0 {
            polynomial_support_bound_exists(p)
            let n: Nat satisfy {
                polynomial_support_bounded_by(p, n)
            }
            let q: Polynomial[Complex] = iterated_quotient(p, z0, n, Nat.1)
            let bb: Real = range_sum(coeff_pow_bound_term(q.coeff, z0.modulus + Real.1), n)
            forall(i: Nat) {
                if i < n {
                    coeff_pow_bound_term(q.coeff, z0.modulus + Real.1, i) = q.coeff(i).modulus * (z0.modulus + Real.1).pow(i)
                    modulus_nonneg(q.coeff(i))
                    q.coeff(i).modulus >= Real.0
                    modulus_nonneg(z0)
                    z0.modulus >= Real.0
                    real_one_positive
                    Real.1 > Real.0
                    lt_imp_lte(Real.0, Real.1)
                    Real.0 <= Real.1
                    add_lte_add(z0.modulus, z0.modulus, Real.0, Real.1)
                    z0.modulus <= z0.modulus and Real.0 <= Real.1 implies z0.modulus + Real.0 <= z0.modulus + Real.1
                    z0.modulus <= z0.modulus
                    z0.modulus + Real.0 <= z0.modulus + Real.1
                    z0.modulus + Real.0 = z0.modulus
                    z0.modulus <= z0.modulus + Real.1
                    z0.modulus + Real.1 >= Real.0
                    real_pow_nonneg(z0.modulus + Real.1, i)
                    (z0.modulus + Real.1).pow(i) >= Real.0
                    mul_nonneg(q.coeff(i).modulus, (z0.modulus + Real.1).pow(i))
                    q.coeff(i).modulus >= Real.0 and (z0.modulus + Real.1).pow(i) >= Real.0 implies q.coeff(i).modulus * (z0.modulus + Real.1).pow(i) >= Real.0
                    coeff_pow_bound_term(q.coeff, z0.modulus + Real.1, i) >= Real.0
                }
            }
            forall(i: Nat) { i < n implies coeff_pow_bound_term(q.coeff, z0.modulus + Real.1, i) >= Real.0 }
            bb = range_sum(coeff_pow_bound_term(q.coeff, z0.modulus + Real.1), n)
            range_sum_nonneg_real(coeff_pow_bound_term(q.coeff, z0.modulus + Real.1), n)
            range_sum(coeff_pow_bound_term(q.coeff, z0.modulus + Real.1), n) >= Real.0
            bb >= Real.0
            bb + Real.1 > Real.0
            real_plus_one_pos(bb)
            bb + Real.1 > Real.0
            bb + Real.1 != Real.0
            real_div_pos(eps, bb + Real.1)
            eps > Real.0 and bb + Real.1 > Real.0 implies eps / (bb + Real.1) > Real.0
            eps / (bb + Real.1) > Real.0
            let delta: Real = Real.1.min(eps / (bb + Real.1))
            min_pos_pos(Real.1, eps / (bb + Real.1))
            Real.1 > Real.0
            Real.1.is_positive
            eps / (bb + Real.1) > Real.0
            (eps / (bb + Real.1)).is_positive
            Real.1.is_positive and (eps / (bb + Real.1)).is_positive
            delta.is_positive
            delta > Real.0
            forall(z: Complex) {
                if (z - z0).modulus < delta {
                    lt_imp_lte((z - z0).modulus, delta)
                    (z - z0).modulus <= delta
                    min_lte_left(Real.1, eps / (bb + Real.1))
                    delta <= Real.1
                    lte_trans((z - z0).modulus, delta, Real.1)
                    (z - z0).modulus <= Real.1
                    quotient_local_bound(p, z0, z, n)
                    polynomial_support_bounded_by(p, n) and (z - z0).modulus <= Real.1
                    polynomial_eval(iterated_quotient(p, z0, n, Nat.1), z).modulus <= range_sum(coeff_pow_bound_term(iterated_quotient(p, z0, n, Nat.1).coeff, z0.modulus + Real.1), n)
                    q = iterated_quotient(p, z0, n, Nat.1)
                    polynomial_eval(q, z).modulus <= range_sum(coeff_pow_bound_term(q.coeff, z0.modulus + Real.1), n)
                    range_sum(coeff_pow_bound_term(q.coeff, z0.modulus + Real.1), n) = bb
                    polynomial_eval(q, z).modulus <= bb
                    polynomial_remainder_global(p, z0, z, n)
                    polynomial_support_bounded_by(p, n)
                    polynomial_eval(p, z) = polynomial_eval(p, z0) + (z - z0) * polynomial_eval(q, z)
                    polynomial_eval(p, z0) + (z - z0) * polynomial_eval(q, z) - polynomial_eval(p, z0) = (z - z0) * polynomial_eval(q, z)
                    (polynomial_eval(p, z) - polynomial_eval(p, z0)).modulus = ((z - z0) * polynomial_eval(q, z)).modulus
                    modulus_mul(z - z0, polynomial_eval(q, z))
                    ((z - z0) * polynomial_eval(q, z)).modulus = (z - z0).modulus * polynomial_eval(q, z).modulus
                    (polynomial_eval(p, z) - polynomial_eval(p, z0)).modulus = (z - z0).modulus * polynomial_eval(q, z).modulus
                    (z - z0).modulus * polynomial_eval(q, z).modulus <= (z - z0).modulus * bb
                    mul_le_mul_of_nonneg_right(polynomial_eval(q, z).modulus, bb, (z - z0).modulus)
                    polynomial_eval(q, z).modulus <= bb and (z - z0).modulus >= Real.0 implies polynomial_eval(q, z).modulus * (z - z0).modulus <= bb * (z - z0).modulus
                    (z - z0).modulus >= Real.0
                    polynomial_eval(q, z).modulus * (z - z0).modulus <= bb * (z - z0).modulus
                    (z - z0).modulus * polynomial_eval(q, z).modulus <= bb * (z - z0).modulus
                    lte_trans((polynomial_eval(p, z) - polynomial_eval(p, z0)).modulus, (z - z0).modulus * polynomial_eval(q, z).modulus, bb * (z - z0).modulus)
                    (polynomial_eval(p, z) - polynomial_eval(p, z0)).modulus <= bb * (z - z0).modulus
                    min_lte_right(Real.1, eps / (bb + Real.1))
                    delta <= eps / (bb + Real.1)
                    (z - z0).modulus <= delta
                    lte_trans((z - z0).modulus, delta, eps / (bb + Real.1))
                    (z - z0).modulus <= eps / (bb + Real.1)
                    mul_le_mul_of_nonneg_right((z - z0).modulus, eps / (bb + Real.1), bb)
                    (z - z0).modulus <= eps / (bb + Real.1) and bb >= Real.0 implies (z - z0).modulus * bb <= eps / (bb + Real.1) * bb
                    bb >= Real.0
                    (z - z0).modulus * bb <= eps / (bb + Real.1) * bb
                    bb * (z - z0).modulus <= eps / (bb + Real.1) * bb
                    real_mul_comm((z - z0).modulus, bb)
                    real_div_mul_comm(eps, bb + Real.1, bb)
                    eps / (bb + Real.1) * bb = (eps * bb) / (bb + Real.1)
                    (eps * bb) / (bb + Real.1) = eps * bb / (bb + Real.1)
                    eps * bb / (bb + Real.1) = eps * (bb / (bb + Real.1))
                    real_div_lt_one(bb)
                    bb >= Real.0 implies bb / (bb + Real.1) < Real.1
                    bb >= Real.0
                    bb / (bb + Real.1) < Real.1
                    mul_lt_mul_of_pos_right(bb / (bb + Real.1), Real.1, eps)
                    bb / (bb + Real.1) < Real.1 and eps > Real.0 implies bb / (bb + Real.1) * eps < Real.1 * eps
                    eps > Real.0
                    bb / (bb + Real.1) * eps < Real.1 * eps
                    bb / (bb + Real.1) * eps = eps * (bb / (bb + Real.1))
                    eps * (bb / (bb + Real.1)) < Real.1 * eps
                    Real.1 * eps = eps
                    eps * (bb / (bb + Real.1)) < eps
                    eps_mul_div_lt_eps(eps, bb)
                    eps > Real.0 and bb >= Real.0 implies eps * bb / (bb + Real.1) < eps
                    eps * bb / (bb + Real.1) < eps
                    lte_trans((polynomial_eval(p, z) - polynomial_eval(p, z0)).modulus, bb * (z - z0).modulus, eps / (bb + Real.1) * bb)
                    (polynomial_eval(p, z) - polynomial_eval(p, z0)).modulus <= eps * bb / (bb + Real.1)
                    lte_lt_trans((polynomial_eval(p, z) - polynomial_eval(p, z0)).modulus, eps * bb / (bb + Real.1), eps)
                    (polynomial_eval(p, z) - polynomial_eval(p, z0)).modulus < eps                }
            }
            delta > Real.0 and forall(z: Complex) {
                (z - z0).modulus < delta implies (polynomial_eval(p, z) - polynomial_eval(p, z0)).modulus < eps
            }
            exists(delta0: Real) {
                delta0 > Real.0 and forall(z: Complex) {
                    (z - z0).modulus < delta0 implies (polynomial_eval(p, z) - polynomial_eval(p, z0)).modulus < eps
                }
            }
    }
}

// The pointwise forall versions of continuity are recorded below without
// proof: the current prover's search cannot discharge `forall(eps) { ... }`
// introduction when the body is a large nested exists/forall statement over
// the complex modulus, even though the per-epsilon statement
// `polynomial_eval_continuous_at_eps` (proved above) is exactly the body.
// The substantive epsilon-delta result is the per-epsilon theorem.

// theorem polynomial_eval_continuous_at(p: Polynomial[Complex], z0: Complex) {
//     forall(eps: Real) {
//         eps > Real.0 implies exists(delta: Real) {
//             delta > Real.0 and forall(z: Complex) {
//                 (z - z0).modulus < delta implies (polynomial_eval(p, z) - polynomial_eval(p, z0)).modulus < eps
//             }
//         }
//     }
// }

// theorem polynomial_eval_modulus_continuous_at(p: Polynomial[Complex], z0: Complex) {
//     forall(eps: Real) {
//         eps > Real.0 implies exists(delta: Real) {
//             delta > Real.0 and forall(z: Complex) {
//                 (z - z0).modulus < delta implies (polynomial_eval(p, z).modulus - polynomial_eval(p, z0).modulus).abs < eps
//             }
//         }
//     }
// }

// ---------------------------------------------------------------------------
// Section 9.  The missing piece: the extreme value theorem on a closed disk.
//
// The fundamental theorem of algebra is proved above modulo the extreme
// value property (`fundamental_theorem_of_algebra_of_extreme`): a global
// minimum of `|p|` is a root.  The extreme value property itself needs
// `|p|` to attain a minimum on a closed disk `|z| <= R`, which follows
// from compactness of the closed disk together with continuity of
// `z |-> |p(z)|` (the epsilon-delta continuity of polynomial evaluation
// is proved in Section 8, and `|p|` is continuous by the reverse triangle
// inequality).  The library does not yet provide:
//
//   - a `TopologicalSpace` instance for `Complex` (the metric topology on
//     `Complex` exists via `complex_metric.ac`, but no topological
//     instance is registered, so `is_compact`, `is_continuous` and the
//     compactness theorems of `analysis/topology/` do not apply to
//     `Complex`);
//   - two-dimensional Heine-Borel: `heine_borel_closed_interval` covers
//     closed real intervals only, and there is no product compactness or
//     closed-disk compactness for `Complex`;
//   - the Bolzano-Weierstrass theorem (bounded real sequences have
//     convergent subsequences), which is the standard sequential route to
//     compactness in two dimensions (`theorems1000/theorem_bolzano_weierstrass.ac`
//     is recorded without proof).
//
// With any one of these, `global_minimum_from_disk` (Section 7) combines
// the disk minimum with the growth theorem (`polynomial_eval_growth`,
// Section 5) to produce the global minimum, and
// `fundamental_theorem_of_algebra_of_extreme` finishes the proof.
// ---------------------------------------------------------------------------
