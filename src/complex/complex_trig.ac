from nat import Nat, mul_suc_right, alt_induction, pow_zero
from rat import Rat
from list import partial, partial_scalar_mul
from algebra.semigroup import mul_fn
from real import Real, two, two_nonzero, sin_term, cos_term, mul_distrib_right, neg_neg,
    real_mul_comm, limit, converges, converges_to, mul_seq, tail_bound,
    converges_to_imp_converges, converges_imp_converges_to, converges_to_unique, sin_term_abs_converges,
    absolutely_converges, absolutely_converges_imp_converges, abs_neg, neg_distrib
from algebra.ring.ring import mul_neg_right, mul_neg_one_left, mul_neg_left, alternating_sign,
    alternating_sign_zero, alternating_sign_suc, alternating_sign_eq_neg_one_pow
from complex.complex import Complex, re_add, im_add, re_sub, im_sub,
    re_i, im_i, re_from_real, im_from_real, re_mul, im_mul, eq_by_components, mul_comm, mul_assoc,
    mul_zero_left, mul_zero_right, mul_one_right, from_real_neg, from_real_one,
    from_real_eq_zero, mul_from_real, real_mul_lifts, real_add_lifts, div_self, mul_inverse,
    add_neg_cancel, add_zero_left
from complex.complex_exp import complex_exp, complex_exp_zero, complex_i_mul_real,
    complex_exp_i_mul_real_eq, real_sub_self, real_mul_one_left,
    real_pow_suc

/// The complex cosine function: z.cos = (exp(i·z) + exp(-i·z)) / 2.
define complex_cos(z: Complex) -> Complex {
    (complex_exp(Complex.i * z) + complex_exp(-(Complex.i * z))) / Complex.from_real(two)
}

/// The complex sine function: z.sin = (exp(i·z) - exp(-i·z)) / (2·i).
define complex_sin(z: Complex) -> Complex {
    (complex_exp(Complex.i * z) - complex_exp(-(Complex.i * z))) / (Complex.from_real(two) * Complex.i)
}

attributes Complex {
    /// The complex cosine function.
    let cos = complex_cos

    /// The complex sine function.
    let sin = complex_sin
}

// ---------------------------------------------------------------------------
// Small helper lemmas for the complex arithmetic of the definitions.
// ---------------------------------------------------------------------------

/// The negation of the complex zero is the complex zero.
theorem complex_neg_zero {
    -Complex.0 = Complex.0
} by {
    add_neg_cancel(Complex.0)
    Complex.0 + -Complex.0 = Complex.0
    add_zero_left(-Complex.0)
    Complex.0 + -Complex.0 = -Complex.0
    -Complex.0 = Complex.0
}

/// Dividing the complex zero by anything yields the complex zero.
theorem complex_zero_div(a: Complex) {
    Complex.0 / a = Complex.0
} by {
    Complex.0 / a = Complex.0 * a.inverse
    mul_zero_left(a.inverse)
    Complex.0 * a.inverse = Complex.0
    Complex.0 / a = Complex.0
}

/// The embedding of the real number two is nonzero.
theorem complex_from_real_two_nonzero {
    Complex.from_real(two) != Complex.0
} by {
    from_real_eq_zero(two)
    Complex.from_real(two) = Complex.0 iff two = Real.0
    two_nonzero
    two != Real.0
    Complex.from_real(two) != Complex.0
}

/// The real part of 2·i is zero.
theorem complex_two_i_re {
    (Complex.from_real(two) * Complex.i).re = Real.0
} by {
    re_mul(Complex.from_real(two), Complex.i)
    (Complex.from_real(two) * Complex.i).re =
        Complex.from_real(two).re * Complex.i.re - Complex.from_real(two).im * Complex.i.im
    re_from_real(two)
    Complex.from_real(two).re = two
    im_from_real(two)
    Complex.from_real(two).im = Real.0
    re_i
    Complex.i.re = Real.0
    im_i
    Complex.i.im = Real.1
    (Complex.from_real(two) * Complex.i).re = two * Real.0 - Real.0 * Real.1
    two * Real.0 = Real.0
    Real.0 * Real.1 = Real.0
    Real.0 - Real.0 = Real.0
    (Complex.from_real(two) * Complex.i).re = Real.0
}

/// The imaginary part of 2·i is two.
theorem complex_two_i_im {
    (Complex.from_real(two) * Complex.i).im = two
} by {
    im_mul(Complex.from_real(two), Complex.i)
    (Complex.from_real(two) * Complex.i).im =
        Complex.from_real(two).re * Complex.i.im + Complex.from_real(two).im * Complex.i.re
    re_from_real(two)
    Complex.from_real(two).re = two
    im_from_real(two)
    Complex.from_real(two).im = Real.0
    re_i
    Complex.i.re = Real.0
    im_i
    Complex.i.im = Real.1
    (Complex.from_real(two) * Complex.i).im = two * Real.1 + Real.0 * Real.0
    two * Real.1 = two
    Real.0 * Real.0 = Real.0
    two + Real.0 = two
    (Complex.from_real(two) * Complex.i).im = two
}

/// The complex number 2·i is nonzero.
theorem complex_two_i_nonzero {
    Complex.from_real(two) * Complex.i != Complex.0
} by {
    if Complex.from_real(two) * Complex.i = Complex.0 {
        complex_two_i_im
        (Complex.from_real(two) * Complex.i).im = two
        Complex.0.im = Real.0
        (Complex.from_real(two) * Complex.i).im = Complex.0.im
        two = Real.0
        two_nonzero
        two != Real.0
        false
    }
}

/// Cancelling a nonzero complex factor on the right of a quotient:
/// (b·a) / b = a.
theorem complex_div_mul_cancel(b: Complex, a: Complex) {
    b != Complex.0 implies (b * a) / b = a
} by {
    if b != Complex.0 {
        (b * a) / b = (b * a) * b.inverse
        mul_comm(b, a)
        b * a = a * b
        (b * a) * b.inverse = (a * b) * b.inverse
        mul_assoc(a, b, b.inverse)
        (a * b) * b.inverse = a * (b * b.inverse)
        mul_inverse(b)
        b != Complex.0 implies b * b.inverse = Complex.1
        b * b.inverse = Complex.1
        a * (b * b.inverse) = a * Complex.1
        mul_one_right(a)
        a * Complex.1 = a
        a * (b * b.inverse) = a
        (b * a) / b = a
    }
}

/// Doubling a real number adds it to itself: 2·x = x + x.
theorem real_two_mul(x: Real) {
    two * x = x + x
} by {
    two = Real.1 + Real.1
    two * x = (Real.1 + Real.1) * x
    mul_distrib_right(Real.1, Real.1, x)
    (Real.1 + Real.1) * x = Real.1 * x + Real.1 * x
    real_mul_one_left(x)
    Real.1 * x = x
    (Real.1 + Real.1) * x = x + x
    two * x = x + x
}

/// Negation of i·x is i·(-x).
theorem complex_i_mul_real_neg(x: Real) {
    -complex_i_mul_real(x) = complex_i_mul_real(-x)
} by {
    complex_i_mul_real(x) = Complex.i * Complex.from_real(x)
    complex_i_mul_real(-x) = Complex.i * Complex.from_real(-x)
    from_real_neg(x)
    Complex.from_real(-x) = -Complex.from_real(x)
    Complex.i * Complex.from_real(-x) = Complex.i * (-Complex.from_real(x))
    mul_neg_right(Complex.i, Complex.from_real(x))
    Complex.i * (-Complex.from_real(x)) = -(Complex.i * Complex.from_real(x))
    Complex.i * Complex.from_real(-x) = -(Complex.i * Complex.from_real(x))
    complex_i_mul_real(-x) = -(Complex.i * Complex.from_real(x))
    -complex_i_mul_real(x) = -(Complex.i * Complex.from_real(x))
    complex_i_mul_real(-x) = -complex_i_mul_real(x)
}

/// Every complex number of the form a + i·b is built from embeddings of a and b.
theorem complex_new_eq_from_real_i(a: Real, b: Real) {
    Complex.new(a, b) = Complex.from_real(a) + Complex.i * Complex.from_real(b)
} by {
    mul_from_real(Complex.i, b)
    Complex.i * Complex.from_real(b) = Complex.new(Complex.i.re * b, Complex.i.im * b)
    re_i
    Complex.i.re = Real.0
    im_i
    Complex.i.im = Real.1
    Complex.new(Complex.i.re * b, Complex.i.im * b) = Complex.new(Real.0 * b, Real.1 * b)
    Real.0 * b = Real.0
    Real.1 * b = b
    Complex.i * Complex.from_real(b) = Complex.new(Real.0, b)
    Complex.from_real(a) = Complex.new(a, Real.0)
    Complex.from_real(a) + Complex.i * Complex.from_real(b) = Complex.new(a, Real.0) + Complex.new(Real.0, b)
    re_add(Complex.new(a, Real.0), Complex.new(Real.0, b))
    (Complex.new(a, Real.0) + Complex.new(Real.0, b)).re = Complex.new(a, Real.0).re + Complex.new(Real.0, b).re
    im_add(Complex.new(a, Real.0), Complex.new(Real.0, b))
    (Complex.new(a, Real.0) + Complex.new(Real.0, b)).im = Complex.new(a, Real.0).im + Complex.new(Real.0, b).im
    Complex.new(a, Real.0).re = a
    Complex.new(a, Real.0).im = Real.0
    Complex.new(Real.0, b).re = Real.0
    Complex.new(Real.0, b).im = b
    (Complex.new(a, Real.0) + Complex.new(Real.0, b)).re = a + Real.0
    (Complex.new(a, Real.0) + Complex.new(Real.0, b)).im = Real.0 + b
    a + Real.0 = a
    Real.0 + b = b
    (Complex.new(a, Real.0) + Complex.new(Real.0, b)).re = a
    (Complex.new(a, Real.0) + Complex.new(Real.0, b)).im = b
    eq_by_components(Complex.new(a, Real.0) + Complex.new(Real.0, b), Complex.new(a, b))
    Complex.new(a, Real.0) + Complex.new(Real.0, b) = Complex.new(a, b)
    Complex.from_real(a) + Complex.i * Complex.from_real(b) = Complex.new(a, b)
}

// ---------------------------------------------------------------------------
// Basic values.
// ---------------------------------------------------------------------------

/// The complex cosine of zero is one.
theorem complex_cos_zero {
    complex_cos(Complex.0) = Complex.1
} by {
    complex_cos(Complex.0) =
        (complex_exp(Complex.i * Complex.0) + complex_exp(-(Complex.i * Complex.0))) / Complex.from_real(two)
    mul_zero_right(Complex.i)
    Complex.i * Complex.0 = Complex.0
    -(Complex.i * Complex.0) = -Complex.0
    complex_neg_zero
    -Complex.0 = Complex.0
    complex_exp(Complex.i * Complex.0) = complex_exp(Complex.0)
    complex_exp_zero
    complex_exp(Complex.0) = Complex.1
    complex_exp(Complex.i * Complex.0) = Complex.1
    complex_exp(-(Complex.i * Complex.0)) = complex_exp(-Complex.0)
    complex_exp(-(Complex.i * Complex.0)) = Complex.1
    real_add_lifts(Real.1, Real.1)
    Complex.from_real(Real.1 + Real.1) = Complex.from_real(Real.1) + Complex.from_real(Real.1)
    from_real_one
    Complex.from_real(Real.1) = Complex.1
    Complex.from_real(Real.1 + Real.1) = Complex.1 + Complex.1
    two = Real.1 + Real.1
    Complex.from_real(two) = Complex.from_real(Real.1 + Real.1)
    Complex.from_real(two) = Complex.1 + Complex.1
    complex_exp(Complex.i * Complex.0) + complex_exp(-(Complex.i * Complex.0)) = Complex.1 + Complex.1
    complex_exp(Complex.i * Complex.0) + complex_exp(-(Complex.i * Complex.0)) = Complex.from_real(two)
    complex_cos(Complex.0) = Complex.from_real(two) / Complex.from_real(two)
    complex_from_real_two_nonzero
    Complex.from_real(two) != Complex.0
    div_self(Complex.from_real(two))
    Complex.from_real(two) != Complex.0 implies Complex.from_real(two) / Complex.from_real(two) = Complex.1
    Complex.from_real(two) / Complex.from_real(two) = Complex.1
    complex_cos(Complex.0) = Complex.1
}

/// The complex sine of zero is zero.
theorem complex_sin_zero {
    complex_sin(Complex.0) = Complex.0
} by {
    complex_sin(Complex.0) =
        (complex_exp(Complex.i * Complex.0) - complex_exp(-(Complex.i * Complex.0))) /
        (Complex.from_real(two) * Complex.i)
    mul_zero_right(Complex.i)
    Complex.i * Complex.0 = Complex.0
    -(Complex.i * Complex.0) = -Complex.0
    complex_neg_zero
    -Complex.0 = Complex.0
    complex_exp(Complex.i * Complex.0) = complex_exp(Complex.0)
    complex_exp_zero
    complex_exp(Complex.0) = Complex.1
    complex_exp(Complex.i * Complex.0) = Complex.1
    complex_exp(-(Complex.i * Complex.0)) = complex_exp(-Complex.0)
    complex_exp(-(Complex.i * Complex.0)) = Complex.1
    Complex.1 - Complex.1 = Complex.0
    complex_exp(Complex.i * Complex.0) - complex_exp(-(Complex.i * Complex.0)) = Complex.0
    complex_zero_div(Complex.from_real(two) * Complex.i)
    Complex.0 / (Complex.from_real(two) * Complex.i) = Complex.0
    complex_sin(Complex.0) = Complex.0
}

// ---------------------------------------------------------------------------
// Euler's formula.  The complex exponential of i·x splits into the real
// cosine and sine series (proved in complex_exp.ac as
// complex_exp_i_mul_real_eq), so exp(i·x) = x.cos + i·x.sin follows
// immediately.  Identifying the complex cosine and sine of an embedded real
// number with the embeddings of the real cosine and sine needs the parity
// laws (-x).cos = x.cos and (-x).sin = -x.sin; the real package does not
// export them from its root, so they are reproduced locally below.
// ---------------------------------------------------------------------------

/// Euler's formula: exp(i·x) = x.cos + i·x.sin for an embedded real number x.
theorem complex_euler_formula_real(x: Real) {
    complex_exp(Complex.i * Complex.from_real(x)) =
        Complex.from_real(x.cos) + Complex.i * Complex.from_real(x.sin)
} by {
    complex_i_mul_real(x) = Complex.i * Complex.from_real(x)
    complex_exp(Complex.i * Complex.from_real(x)) = complex_exp(complex_i_mul_real(x))
    complex_exp_i_mul_real_eq(x)
    complex_exp(complex_i_mul_real(x)) = Complex.new(x.cos, x.sin)
    complex_exp(Complex.i * Complex.from_real(x)) = Complex.new(x.cos, x.sin)
    complex_new_eq_from_real_i(x.cos, x.sin)
    Complex.new(x.cos, x.sin) = Complex.from_real(x.cos) + Complex.i * Complex.from_real(x.sin)
    complex_exp(Complex.i * Complex.from_real(x)) =
        Complex.from_real(x.cos) + Complex.i * Complex.from_real(x.sin)
}

// ---------------------------------------------------------------------------
// The parity laws (-x).cos = x.cos and (-x).sin = -x.sin, reproduced from the
// real trigonometric series since the real package does not export them.
// ---------------------------------------------------------------------------

/// Doubling a successor is two more than doubling.
theorem real_two_mul_suc(k: Nat) {
    Nat.2 * k.suc = (Nat.2 * k).suc.suc
} by {
    mul_suc_right(Nat.2, k)
    Nat.2 * k.suc = Nat.2 + Nat.2 * k
    Nat.2 + Nat.2 * k = Nat.2 * k + Nat.2
    Nat.2 * k + Nat.2 = (Nat.2 * k).suc.suc
    Nat.2 * k.suc = (Nat.2 * k).suc.suc
}

/// The alternating sign at an even index is one.
theorem real_alternating_sign_double(n: Nat) {
    alternating_sign[Real](Nat.2 * n) = Real.1
} by {
    define p(k: Nat) -> Bool {
        alternating_sign[Real](Nat.2 * k) = Real.1
    }
    alternating_sign_zero[Real]
    alternating_sign[Real](Nat.0) = Real.1
    Nat.2 * Nat.0 = Nat.0
    alternating_sign[Real](Nat.2 * Nat.0) = Real.1
    p(Nat.0)
    forall(k: Nat) {
        if p(k) {
            alternating_sign[Real](Nat.2 * k) = Real.1
            alternating_sign_suc[Real](Nat.2 * k)
            alternating_sign_suc[Real]((Nat.2 * k).suc)
            alternating_sign[Real]((Nat.2 * k).suc) = -alternating_sign[Real](Nat.2 * k)
            alternating_sign[Real]((Nat.2 * k).suc.suc) = -alternating_sign[Real]((Nat.2 * k).suc)
            alternating_sign[Real]((Nat.2 * k).suc.suc) = --alternating_sign[Real](Nat.2 * k)
            neg_neg(alternating_sign[Real](Nat.2 * k))
            --alternating_sign[Real](Nat.2 * k) = alternating_sign[Real](Nat.2 * k)
            real_two_mul_suc(k)
            Nat.2 * k.suc = (Nat.2 * k).suc.suc
            alternating_sign[Real](Nat.2 * k.suc) = alternating_sign[Real]((Nat.2 * k).suc.suc)
            alternating_sign[Real](Nat.2 * k.suc) = alternating_sign[Real](Nat.2 * k)
            alternating_sign[Real](Nat.2 * k) = Real.1
            alternating_sign[Real](Nat.2 * k.suc) = Real.1
            p(k.suc)
        }
    }
    p(Nat.0) and forall(k: Nat) { p(k) implies p(k.suc) }
    alt_induction(p)
    p(n)
}

/// Powers distribute over multiplication.
theorem real_pow_mul_distrib(x: Real, y: Real, n: Nat) {
    (x * y).pow(n) = x.pow(n) * y.pow(n)
} by {
    define p(k: Nat) -> Bool {
        (x * y).pow(k) = x.pow(k) * y.pow(k)
    }
    pow_zero(x * y)
    (x * y).pow(Nat.0) = Real.1
    pow_zero(x)
    x.pow(Nat.0) = Real.1
    pow_zero(y)
    y.pow(Nat.0) = Real.1
    Real.1 * Real.1 = Real.1
    (x * y).pow(Nat.0) = x.pow(Nat.0) * y.pow(Nat.0)
    p(Nat.0)
    forall(k: Nat) {
        if p(k) {
            (x * y).pow(k) = x.pow(k) * y.pow(k)
            real_pow_suc(x * y, k)
            (x * y).pow(k.suc) = (x * y) * (x * y).pow(k)
            (x * y).pow(k) = x.pow(k) * y.pow(k)
            (x * y).pow(k.suc) = (x * y) * (x.pow(k) * y.pow(k))
            (x * y) * (x.pow(k) * y.pow(k)) = x * y * x.pow(k) * y.pow(k)
            real_mul_comm(y, x.pow(k))
            y * x.pow(k) = x.pow(k) * y
            x * (y * x.pow(k)) * y.pow(k) = x * (x.pow(k) * y) * y.pow(k)
            x * y * x.pow(k) * y.pow(k) = x * (y * x.pow(k)) * y.pow(k)
            x * (x.pow(k) * y) * y.pow(k) = x * x.pow(k) * y * y.pow(k)
            x * y * x.pow(k) * y.pow(k) = x * x.pow(k) * y * y.pow(k)
            real_pow_suc(x, k)
            x.pow(k.suc) = x * x.pow(k)
            real_pow_suc(y, k)
            y.pow(k.suc) = y * y.pow(k)
            x * x.pow(k) * y * y.pow(k) = x.pow(k.suc) * y.pow(k.suc)
            (x * y).pow(k.suc) = x.pow(k.suc) * y.pow(k.suc)
            p(k.suc)
        }
    }
    p(Nat.0) and forall(k: Nat) { p(k) implies p(k.suc) }
    alt_induction(p)
    p(n)
}

/// An even power of -x equals the same power of x.
theorem real_neg_pow_even(x: Real, n: Nat) {
    (-x).pow(Nat.2 * n) = x.pow(Nat.2 * n)
} by {
    mul_neg_one_left(x)
    -Real.1 * x = -x
    (-x).pow(Nat.2 * n) = (-Real.1 * x).pow(Nat.2 * n)
    real_pow_mul_distrib(-Real.1, x, Nat.2 * n)
    (-Real.1 * x).pow(Nat.2 * n) = (-Real.1).pow(Nat.2 * n) * x.pow(Nat.2 * n)
    alternating_sign_eq_neg_one_pow[Real](Nat.2 * n)
    alternating_sign[Real](Nat.2 * n) = (-Real.1).pow(Nat.2 * n)
    (-Real.1).pow(Nat.2 * n) = alternating_sign[Real](Nat.2 * n)
    real_alternating_sign_double(n)
    alternating_sign[Real](Nat.2 * n) = Real.1
    (-Real.1).pow(Nat.2 * n) = Real.1
    (-Real.1 * x).pow(Nat.2 * n) = Real.1 * x.pow(Nat.2 * n)
    Real.1 * x.pow(Nat.2 * n) = x.pow(Nat.2 * n)
    (-x).pow(Nat.2 * n) = x.pow(Nat.2 * n)
}

/// The cosine terms are even.
theorem real_cos_term_neg(x: Real, n: Nat) {
    cos_term(-x, n) = cos_term(x, n)
} by {
    real_neg_pow_even(x, n)
    (-x).pow(Nat.2 * n) = x.pow(Nat.2 * n)
    cos_term(-x, n) = alternating_sign[Real](n) * (-x).pow(Nat.2 * n) / Real.from_rat(Rat.from_nat((Nat.2 * n).factorial))
    alternating_sign[Real](n) * (-x).pow(Nat.2 * n) / Real.from_rat(Rat.from_nat((Nat.2 * n).factorial)) =
        alternating_sign[Real](n) * x.pow(Nat.2 * n) / Real.from_rat(Rat.from_nat((Nat.2 * n).factorial))
    cos_term(-x, n) = cos_term(x, n)
}

/// Cosine is even.
theorem real_cos_neg(x: Real) {
    (-x).cos = x.cos
} by {
    forall(n: Nat) {
        real_cos_term_neg(x, n)
        cos_term(-x, n) = cos_term(x, n)
    }
    cos_term(-x) = cos_term(x)
    partial(cos_term(-x)) = partial(cos_term(x))
    limit(partial(cos_term(-x))) = limit(partial(cos_term(x)))
    (-x).cos = limit(partial(cos_term(-x)))
    (-x).cos = limit(partial(cos_term(x)))
    x.cos = limit(partial(cos_term(x)))
    (-x).cos = x.cos
}

// The odd parity law (-x).sin = -x.sin needs the limit of the negated series
// (limit_neg_seq), which the real package does not export.  It is reproduced
// below; if it resists, the theorems depending on it are commented out.

/// The alternating sign at an odd index is negative one.
theorem real_alternating_sign_double_suc(n: Nat) {
    alternating_sign[Real](Nat.2 * n + Nat.1) = -Real.1
} by {
    define p(k: Nat) -> Bool {
        alternating_sign[Real](Nat.2 * k + Nat.1) = -Real.1
    }
    alternating_sign_zero[Real]
    alternating_sign[Real](Nat.0) = Real.1
    alternating_sign_suc[Real](Nat.0)
    alternating_sign[Real](Nat.1) = -alternating_sign[Real](Nat.0)
    alternating_sign[Real](Nat.1) = -Real.1
    Nat.2 * Nat.0 + Nat.1 = Nat.1
    alternating_sign[Real](Nat.2 * Nat.0 + Nat.1) = -Real.1
    p(Nat.0)
    forall(k: Nat) {
        if p(k) {
            alternating_sign[Real](Nat.2 * k + Nat.1) = -Real.1
            alternating_sign_suc[Real](Nat.2 * k + Nat.1)
            alternating_sign_suc[Real]((Nat.2 * k + Nat.1).suc)
            alternating_sign[Real]((Nat.2 * k + Nat.1).suc) = -alternating_sign[Real](Nat.2 * k + Nat.1)
            alternating_sign[Real]((Nat.2 * k + Nat.1).suc.suc) = -alternating_sign[Real]((Nat.2 * k + Nat.1).suc)
            alternating_sign[Real]((Nat.2 * k + Nat.1).suc.suc) = --alternating_sign[Real](Nat.2 * k + Nat.1)
            neg_neg(alternating_sign[Real](Nat.2 * k + Nat.1))
            --alternating_sign[Real](Nat.2 * k + Nat.1) = alternating_sign[Real](Nat.2 * k + Nat.1)
            Nat.2 * k.suc + Nat.1 = (Nat.2 * k + Nat.1).suc.suc
            alternating_sign[Real](Nat.2 * k.suc + Nat.1) = alternating_sign[Real]((Nat.2 * k + Nat.1).suc.suc)
            alternating_sign[Real](Nat.2 * k.suc + Nat.1) = alternating_sign[Real](Nat.2 * k + Nat.1)
            alternating_sign[Real](Nat.2 * k + Nat.1) = -Real.1
            alternating_sign[Real](Nat.2 * k.suc + Nat.1) = -Real.1
            p(k.suc)
        }
    }
    p(Nat.0) and forall(k: Nat) { p(k) implies p(k.suc) }
    alt_induction(p)
    p(n)
}

/// An odd power of -x is the negation of the same power of x.
theorem real_neg_pow_odd(x: Real, n: Nat) {
    (-x).pow(Nat.2 * n + Nat.1) = -x.pow(Nat.2 * n + Nat.1)
} by {
    mul_neg_one_left(x)
    -Real.1 * x = -x
    (-x).pow(Nat.2 * n + Nat.1) = (-Real.1 * x).pow(Nat.2 * n + Nat.1)
    real_pow_mul_distrib(-Real.1, x, Nat.2 * n + Nat.1)
    (-Real.1 * x).pow(Nat.2 * n + Nat.1) = (-Real.1).pow(Nat.2 * n + Nat.1) * x.pow(Nat.2 * n + Nat.1)
    alternating_sign_eq_neg_one_pow[Real](Nat.2 * n + Nat.1)
    alternating_sign[Real](Nat.2 * n + Nat.1) = (-Real.1).pow(Nat.2 * n + Nat.1)
    (-Real.1).pow(Nat.2 * n + Nat.1) = alternating_sign[Real](Nat.2 * n + Nat.1)
    real_alternating_sign_double_suc(n)
    alternating_sign[Real](Nat.2 * n + Nat.1) = -Real.1
    (-Real.1).pow(Nat.2 * n + Nat.1) = -Real.1
    (-Real.1 * x).pow(Nat.2 * n + Nat.1) = (-Real.1) * x.pow(Nat.2 * n + Nat.1)
    mul_neg_one_left(x.pow(Nat.2 * n + Nat.1))
    (-Real.1) * x.pow(Nat.2 * n + Nat.1) = -x.pow(Nat.2 * n + Nat.1)
    (-x).pow(Nat.2 * n + Nat.1) = -x.pow(Nat.2 * n + Nat.1)
}

/// Division commutes with negation in the numerator.
theorem real_neg_div(a: Real, b: Real) {
    (-a) / b = -(a / b)
} by {
    (-a) / b = (-a) * b.inverse
    mul_neg_left(a, b.inverse)
    (-a) * b.inverse = -(a * b.inverse)
    -(a * b.inverse) = -(a / b)
    (-a) / b = -(a / b)
}

/// The sine terms are odd.
theorem real_sin_term_neg(x: Real, n: Nat) {
    sin_term(-x, n) = -sin_term(x, n)
} by {
    real_neg_pow_odd(x, n)
    (-x).pow(Nat.2 * n + Nat.1) = -x.pow(Nat.2 * n + Nat.1)
    sin_term(-x, n) = alternating_sign[Real](n) * (-x).pow(Nat.2 * n + Nat.1) / Real.from_rat(Rat.from_nat((Nat.2 * n + Nat.1).factorial))
    alternating_sign[Real](n) * (-x).pow(Nat.2 * n + Nat.1) / Real.from_rat(Rat.from_nat((Nat.2 * n + Nat.1).factorial)) =
        alternating_sign[Real](n) * (-x.pow(Nat.2 * n + Nat.1)) / Real.from_rat(Rat.from_nat((Nat.2 * n + Nat.1).factorial))
    mul_neg_right(alternating_sign[Real](n), x.pow(Nat.2 * n + Nat.1))
    alternating_sign[Real](n) * -x.pow(Nat.2 * n + Nat.1) = -(alternating_sign[Real](n) * x.pow(Nat.2 * n + Nat.1))
    alternating_sign[Real](n) * (-x.pow(Nat.2 * n + Nat.1)) / Real.from_rat(Rat.from_nat((Nat.2 * n + Nat.1).factorial)) =
        -(alternating_sign[Real](n) * x.pow(Nat.2 * n + Nat.1)) / Real.from_rat(Rat.from_nat((Nat.2 * n + Nat.1).factorial))
    real_neg_div(alternating_sign[Real](n) * x.pow(Nat.2 * n + Nat.1), Real.from_rat(Rat.from_nat((Nat.2 * n + Nat.1).factorial)))
    (-(alternating_sign[Real](n) * x.pow(Nat.2 * n + Nat.1))) / Real.from_rat(Rat.from_nat((Nat.2 * n + Nat.1).factorial)) =
        -((alternating_sign[Real](n) * x.pow(Nat.2 * n + Nat.1)) / Real.from_rat(Rat.from_nat((Nat.2 * n + Nat.1).factorial)))
    -((alternating_sign[Real](n) * x.pow(Nat.2 * n + Nat.1)) / Real.from_rat(Rat.from_nat((Nat.2 * n + Nat.1).factorial))) =
        -sin_term(x, n)
    sin_term(-x, n) = -sin_term(x, n)
}

/// Negation commutes with partial sums.
theorem real_partial_neg_seq(f: Nat -> Real, n: Nat) {
    partial(mul_seq(-Real.1, f), n) = -partial(f, n)
} by {
    partial_scalar_mul(-Real.1, f, n)
    -Real.1 * partial(f, n) = partial(mul_fn(-Real.1, f), n)
    forall(m: Nat) {
        mul_seq(-Real.1, f, m) = -Real.1 * f(m)
        mul_fn(-Real.1, f, m) = -Real.1 * f(m)
        mul_seq(-Real.1, f, m) = mul_fn(-Real.1, f, m)
    }
    mul_seq(-Real.1, f) = mul_fn(-Real.1, f)
    partial(mul_seq(-Real.1, f), n) = partial(mul_fn(-Real.1, f), n)
    partial(mul_fn(-Real.1, f), n) = -Real.1 * partial(f, n)
    mul_neg_one_left(partial(f, n))
    -Real.1 * partial(f, n) = -partial(f, n)
    partial(mul_seq(-Real.1, f), n) = -partial(f, n)
}

/// A negated convergent sequence converges to the negation of the limit.
theorem real_neg_seq_converges_to(a: Nat -> Real) {
    converges(a) implies converges_to(mul_seq(-Real.1, a), -limit(a))
} by {
    if converges(a) {
        converges_imp_converges_to(a)
        converges_to(a, limit(a))
        forall(eps: Real) {
            if eps.is_positive {
                exists(n: Nat) {
                    tail_bound(a, limit(a), n, eps)
                }
                let n: Nat satisfy {
                    tail_bound(a, limit(a), n, eps)
                }
                forall(i: Nat) {
                    if n <= i {
                        tail_bound(a, limit(a), n, eps) = forall(j: Nat) {
                            n <= j implies a(j).is_close(limit(a), eps)
                        }
                        forall(j: Nat) {
                            n <= j implies a(j).is_close(limit(a), eps)
                        }
                        n <= i implies a(i).is_close(limit(a), eps)
                        a(i).is_close(limit(a), eps)
                        a(i).is_close(limit(a), eps) = (a(i) - limit(a)).abs < eps
                        (a(i) - limit(a)).abs < eps
                        mul_seq(-Real.1, a, i) = -Real.1 * a(i)
                        mul_neg_one_left(a(i))
                        -Real.1 * a(i) = -a(i)
                        mul_seq(-Real.1, a, i) = -a(i)
                        mul_seq(-Real.1, a, i) - (-limit(a)) = -a(i) + limit(a)
                        neg_distrib(a(i), -limit(a))
                        -(a(i) + -limit(a)) = -a(i) + -(-limit(a))
                        -(-limit(a)) = limit(a)
                        -a(i) + -(-limit(a)) = -a(i) + limit(a)
                        -(a(i) + -limit(a)) = -a(i) + limit(a)
                        a(i) - limit(a) = a(i) + -limit(a)
                        -(a(i) - limit(a)) = -a(i) + limit(a)
                        -a(i) + limit(a) = -(a(i) - limit(a))
                        mul_seq(-Real.1, a, i) - (-limit(a)) = -(a(i) - limit(a))
                        abs_neg(a(i) - limit(a))
                        (-(a(i) - limit(a))).abs = (a(i) - limit(a)).abs
                        (mul_seq(-Real.1, a, i) - (-limit(a))).abs = (a(i) - limit(a)).abs
                        (mul_seq(-Real.1, a, i) - (-limit(a))).abs < eps
                        mul_seq(-Real.1, a, i).is_close(-limit(a), eps) =
                            (mul_seq(-Real.1, a, i) - (-limit(a))).abs < eps
                        mul_seq(-Real.1, a, i).is_close(-limit(a), eps)
                    }
                }
                tail_bound(mul_seq(-Real.1, a), -limit(a), n, eps)
                exists(n0: Nat) {
                    tail_bound(mul_seq(-Real.1, a), -limit(a), n0, eps)
                }
            }
        }
        forall(eps: Real) {
            eps.is_positive implies exists(n0: Nat) {
                tail_bound(mul_seq(-Real.1, a), -limit(a), n0, eps)
            }
        }
        converges_to(mul_seq(-Real.1, a), -limit(a)) = forall(eps: Real) {
            eps.is_positive implies exists(n0: Nat) {
                tail_bound(mul_seq(-Real.1, a), -limit(a), n0, eps)
            }
        }
        if not converges_to(mul_seq(-Real.1, a), -limit(a)) {
            not forall(eps: Real) {
                eps.is_positive implies exists(n0: Nat) {
                    tail_bound(mul_seq(-Real.1, a), -limit(a), n0, eps)
                }
            }
            let bad_eps: Real satisfy {
                bad_eps.is_positive and forall(n0: Nat) {
                    not tail_bound(mul_seq(-Real.1, a), -limit(a), n0, bad_eps)
                }
            }
            bad_eps.is_positive
            forall(n0: Nat) {
                not tail_bound(mul_seq(-Real.1, a), -limit(a), n0, bad_eps)
            }
            bad_eps.is_positive implies exists(n0: Nat) {
                tail_bound(mul_seq(-Real.1, a), -limit(a), n0, bad_eps)
            }
            exists(n0: Nat) {
                tail_bound(mul_seq(-Real.1, a), -limit(a), n0, bad_eps)
            }
            let w: Nat satisfy {
                tail_bound(mul_seq(-Real.1, a), -limit(a), w, bad_eps)
            }
            not tail_bound(mul_seq(-Real.1, a), -limit(a), w, bad_eps)
            false
        }
        converges_to(mul_seq(-Real.1, a), -limit(a))
    }
}

/// The limit of a negated convergent sequence is the negation of the limit.
theorem real_limit_neg_seq(a: Nat -> Real) {
    converges(a) implies limit(mul_seq(-Real.1, a)) = -limit(a)
} by {
    if converges(a) {
        real_neg_seq_converges_to(a)
        converges_to(mul_seq(-Real.1, a), -limit(a))
        converges_to_imp_converges(mul_seq(-Real.1, a), -limit(a))
        converges(mul_seq(-Real.1, a))
        converges_imp_converges_to(mul_seq(-Real.1, a))
        converges_to(mul_seq(-Real.1, a), limit(mul_seq(-Real.1, a)))
        converges_to_unique(mul_seq(-Real.1, a), -limit(a), limit(mul_seq(-Real.1, a)))
        limit(mul_seq(-Real.1, a)) = -limit(a)
    }
}

/// Sine is odd.
theorem real_sin_neg(x: Real) {
    (-x).sin = -x.sin
} by {
    forall(n: Nat) {
        real_sin_term_neg(x, n)
        sin_term(-x, n) = -sin_term(x, n)
        mul_neg_one_left(sin_term(x, n))
        -Real.1 * sin_term(x, n) = -sin_term(x, n)
        mul_seq(-Real.1, sin_term(x), n) = -Real.1 * sin_term(x, n)
        sin_term(-x, n) = mul_seq(-Real.1, sin_term(x), n)
    }
    sin_term(-x) = mul_seq(-Real.1, sin_term(x))
    partial(sin_term(-x)) = partial(mul_seq(-Real.1, sin_term(x)))
    forall(n: Nat) {
        real_partial_neg_seq(sin_term(x), n)
        partial(mul_seq(-Real.1, sin_term(x)), n) = -partial(sin_term(x), n)
        mul_seq(-Real.1, partial(sin_term(x)), n) = -Real.1 * partial(sin_term(x), n)
        mul_neg_one_left(partial(sin_term(x), n))
        -Real.1 * partial(sin_term(x), n) = -partial(sin_term(x), n)
        mul_seq(-Real.1, partial(sin_term(x)), n) = -partial(sin_term(x), n)
        partial(mul_seq(-Real.1, sin_term(x)), n) = mul_seq(-Real.1, partial(sin_term(x)), n)
    }
    partial(mul_seq(-Real.1, sin_term(x))) = mul_seq(-Real.1, partial(sin_term(x)))
    partial(sin_term(-x)) = mul_seq(-Real.1, partial(sin_term(x)))
    sin_term_abs_converges(x)
    absolutely_converges(sin_term(x))
    absolutely_converges_imp_converges(sin_term(x))
    converges(partial(sin_term(x)))
    real_limit_neg_seq(partial(sin_term(x)))
    converges(partial(sin_term(x))) implies limit(mul_seq(-Real.1, partial(sin_term(x)))) = -limit(partial(sin_term(x)))
    limit(mul_seq(-Real.1, partial(sin_term(x)))) = -limit(partial(sin_term(x)))
    limit(partial(sin_term(-x))) = limit(mul_seq(-Real.1, partial(sin_term(x))))
    limit(partial(sin_term(-x))) = -limit(partial(sin_term(x)))
    (-x).sin = limit(partial(sin_term(-x)))
    (-x).sin = -limit(partial(sin_term(x)))
    x.sin = limit(partial(sin_term(x)))
    (-x).sin = -x.sin
}

// ---------------------------------------------------------------------------
// The complex cosine and sine of an embedded real number are the embeddings
// of the real cosine and sine, using the parity laws above.
// ---------------------------------------------------------------------------

/// The exponential of -i·x is x.cos - i·x.sin.
theorem complex_exp_neg_i_mul_real(x: Real) {
    complex_exp(-complex_i_mul_real(x)) = Complex.new(x.cos, -x.sin)
} by {
    complex_i_mul_real_neg(x)
    -complex_i_mul_real(x) = complex_i_mul_real(-x)
    complex_exp(-complex_i_mul_real(x)) = complex_exp(complex_i_mul_real(-x))
    complex_exp_i_mul_real_eq(-x)
    complex_exp(complex_i_mul_real(-x)) = Complex.new((-x).cos, (-x).sin)
    real_cos_neg(x)
    (-x).cos = x.cos
    real_sin_neg(x)
    (-x).sin = -x.sin
    Complex.new((-x).cos, (-x).sin) = Complex.new(x.cos, -x.sin)
    complex_exp(-complex_i_mul_real(x)) = Complex.new(x.cos, -x.sin)
}

/// The complex cosine of an embedded real number is the embedding of the real cosine.
theorem complex_cos_from_real(x: Real) {
    complex_cos(Complex.from_real(x)) = Complex.from_real(x.cos)
} by {
    complex_cos(Complex.from_real(x)) =
        (complex_exp(Complex.i * Complex.from_real(x)) + complex_exp(-(Complex.i * Complex.from_real(x)))) /
        Complex.from_real(two)

    complex_i_mul_real(x) = Complex.i * Complex.from_real(x)
    complex_exp(Complex.i * Complex.from_real(x)) = complex_exp(complex_i_mul_real(x))
    complex_exp_i_mul_real_eq(x)
    complex_exp(complex_i_mul_real(x)) = Complex.new(x.cos, x.sin)
    complex_exp(Complex.i * Complex.from_real(x)) = Complex.new(x.cos, x.sin)

    complex_i_mul_real_neg(x)
    -complex_i_mul_real(x) = complex_i_mul_real(-x)
    -complex_i_mul_real(x) = -(Complex.i * Complex.from_real(x))
    complex_exp(-(Complex.i * Complex.from_real(x))) = complex_exp(-complex_i_mul_real(x))
    complex_exp_neg_i_mul_real(x)
    complex_exp(-complex_i_mul_real(x)) = Complex.new(x.cos, -x.sin)
    complex_exp(-(Complex.i * Complex.from_real(x))) = Complex.new(x.cos, -x.sin)

    complex_exp(Complex.i * Complex.from_real(x)) + complex_exp(-(Complex.i * Complex.from_real(x))) =
        Complex.new(x.cos, x.sin) + Complex.new(x.cos, -x.sin)
    re_add(Complex.new(x.cos, x.sin), Complex.new(x.cos, -x.sin))
    (Complex.new(x.cos, x.sin) + Complex.new(x.cos, -x.sin)).re =
        Complex.new(x.cos, x.sin).re + Complex.new(x.cos, -x.sin).re
    im_add(Complex.new(x.cos, x.sin), Complex.new(x.cos, -x.sin))
    (Complex.new(x.cos, x.sin) + Complex.new(x.cos, -x.sin)).im =
        Complex.new(x.cos, x.sin).im + Complex.new(x.cos, -x.sin).im
    Complex.new(x.cos, x.sin).re = x.cos
    Complex.new(x.cos, x.sin).im = x.sin
    Complex.new(x.cos, -x.sin).re = x.cos
    Complex.new(x.cos, -x.sin).im = -x.sin
    (Complex.new(x.cos, x.sin) + Complex.new(x.cos, -x.sin)).re = x.cos + x.cos
    (Complex.new(x.cos, x.sin) + Complex.new(x.cos, -x.sin)).im = x.sin + -x.sin
    real_two_mul(x.cos)
    two * x.cos = x.cos + x.cos
    real_sub_self(x.sin)
    x.sin - x.sin = Real.0
    x.sin + -x.sin = Real.0
    Complex.new(x.cos + x.cos, x.sin + -x.sin) = Complex.new(two * x.cos, Real.0)
    eq_by_components(Complex.new(x.cos, x.sin) + Complex.new(x.cos, -x.sin), Complex.new(two * x.cos, Real.0))
    Complex.new(x.cos, x.sin) + Complex.new(x.cos, -x.sin) = Complex.new(two * x.cos, Real.0)
    complex_exp(Complex.i * Complex.from_real(x)) + complex_exp(-(Complex.i * Complex.from_real(x))) =
        Complex.new(two * x.cos, Real.0)
    Complex.new(two * x.cos, Real.0) = Complex.from_real(two * x.cos)
    complex_exp(Complex.i * Complex.from_real(x)) + complex_exp(-(Complex.i * Complex.from_real(x))) =
        Complex.from_real(two * x.cos)
    real_mul_lifts(two, x.cos)
    Complex.from_real(two * x.cos) = Complex.from_real(two) * Complex.from_real(x.cos)
    complex_exp(Complex.i * Complex.from_real(x)) + complex_exp(-(Complex.i * Complex.from_real(x))) =
        Complex.from_real(two) * Complex.from_real(x.cos)

    complex_cos(Complex.from_real(x)) =
        (Complex.from_real(two) * Complex.from_real(x.cos)) / Complex.from_real(two)
    complex_from_real_two_nonzero
    Complex.from_real(two) != Complex.0
    complex_div_mul_cancel(Complex.from_real(two), Complex.from_real(x.cos))
    Complex.from_real(two) != Complex.0 implies (Complex.from_real(two) * Complex.from_real(x.cos)) / Complex.from_real(two) = Complex.from_real(x.cos)
    (Complex.from_real(two) * Complex.from_real(x.cos)) / Complex.from_real(two) = Complex.from_real(x.cos)
    complex_cos(Complex.from_real(x)) = Complex.from_real(x.cos)
}

/// The complex sine of an embedded real number is the embedding of the real sine.
theorem complex_sin_from_real(x: Real) {
    complex_sin(Complex.from_real(x)) = Complex.from_real(x.sin)
} by {
    complex_sin(Complex.from_real(x)) =
        (complex_exp(Complex.i * Complex.from_real(x)) - complex_exp(-(Complex.i * Complex.from_real(x)))) /
        (Complex.from_real(two) * Complex.i)

    complex_i_mul_real(x) = Complex.i * Complex.from_real(x)
    complex_exp(Complex.i * Complex.from_real(x)) = complex_exp(complex_i_mul_real(x))
    complex_exp_i_mul_real_eq(x)
    complex_exp(complex_i_mul_real(x)) = Complex.new(x.cos, x.sin)
    complex_exp(Complex.i * Complex.from_real(x)) = Complex.new(x.cos, x.sin)

    complex_i_mul_real_neg(x)
    -complex_i_mul_real(x) = complex_i_mul_real(-x)
    -complex_i_mul_real(x) = -(Complex.i * Complex.from_real(x))
    complex_exp(-(Complex.i * Complex.from_real(x))) = complex_exp(-complex_i_mul_real(x))
    complex_exp_neg_i_mul_real(x)
    complex_exp(-complex_i_mul_real(x)) = Complex.new(x.cos, -x.sin)
    complex_exp(-(Complex.i * Complex.from_real(x))) = Complex.new(x.cos, -x.sin)

    complex_exp(Complex.i * Complex.from_real(x)) - complex_exp(-(Complex.i * Complex.from_real(x))) =
        Complex.new(x.cos, x.sin) - Complex.new(x.cos, -x.sin)
    re_sub(Complex.new(x.cos, x.sin), Complex.new(x.cos, -x.sin))
    (Complex.new(x.cos, x.sin) - Complex.new(x.cos, -x.sin)).re =
        Complex.new(x.cos, x.sin).re - Complex.new(x.cos, -x.sin).re
    im_sub(Complex.new(x.cos, x.sin), Complex.new(x.cos, -x.sin))
    (Complex.new(x.cos, x.sin) - Complex.new(x.cos, -x.sin)).im =
        Complex.new(x.cos, x.sin).im - Complex.new(x.cos, -x.sin).im
    Complex.new(x.cos, x.sin).re = x.cos
    Complex.new(x.cos, x.sin).im = x.sin
    Complex.new(x.cos, -x.sin).re = x.cos
    Complex.new(x.cos, -x.sin).im = -x.sin
    (Complex.new(x.cos, x.sin) - Complex.new(x.cos, -x.sin)).re = x.cos - x.cos
    (Complex.new(x.cos, x.sin) - Complex.new(x.cos, -x.sin)).im = x.sin - (-x.sin)
    real_sub_self(x.cos)
    x.cos - x.cos = Real.0
    neg_neg(x.sin)
    -(-x.sin) = x.sin
    x.sin - (-x.sin) = x.sin + x.sin
    real_two_mul(x.sin)
    two * x.sin = x.sin + x.sin
    x.sin - (-x.sin) = two * x.sin
    Complex.new(x.cos - x.cos, x.sin - (-x.sin)) = Complex.new(Real.0, two * x.sin)
    eq_by_components(Complex.new(x.cos, x.sin) - Complex.new(x.cos, -x.sin), Complex.new(Real.0, two * x.sin))
    Complex.new(x.cos, x.sin) - Complex.new(x.cos, -x.sin) = Complex.new(Real.0, two * x.sin)
    complex_exp(Complex.i * Complex.from_real(x)) - complex_exp(-(Complex.i * Complex.from_real(x))) =
        Complex.new(Real.0, two * x.sin)

    mul_from_real(Complex.from_real(two) * Complex.i, x.sin)
    (Complex.from_real(two) * Complex.i) * Complex.from_real(x.sin) =
        Complex.new((Complex.from_real(two) * Complex.i).re * x.sin,
            (Complex.from_real(two) * Complex.i).im * x.sin)
    complex_two_i_re
    (Complex.from_real(two) * Complex.i).re = Real.0
    complex_two_i_im
    (Complex.from_real(two) * Complex.i).im = two
    (Complex.from_real(two) * Complex.i) * Complex.from_real(x.sin) =
        Complex.new(Real.0 * x.sin, two * x.sin)
    Real.0 * x.sin = Real.0
    (Complex.from_real(two) * Complex.i) * Complex.from_real(x.sin) = Complex.new(Real.0, two * x.sin)
    complex_exp(Complex.i * Complex.from_real(x)) - complex_exp(-(Complex.i * Complex.from_real(x))) =
        (Complex.from_real(two) * Complex.i) * Complex.from_real(x.sin)

    complex_sin(Complex.from_real(x)) =
        ((Complex.from_real(two) * Complex.i) * Complex.from_real(x.sin)) / (Complex.from_real(two) * Complex.i)
    complex_two_i_nonzero
    Complex.from_real(two) * Complex.i != Complex.0
    complex_div_mul_cancel(Complex.from_real(two) * Complex.i, Complex.from_real(x.sin))
    Complex.from_real(two) * Complex.i != Complex.0 implies ((Complex.from_real(two) * Complex.i) * Complex.from_real(x.sin)) / (Complex.from_real(two) * Complex.i) = Complex.from_real(x.sin)
    ((Complex.from_real(two) * Complex.i) * Complex.from_real(x.sin)) /
        (Complex.from_real(two) * Complex.i) = Complex.from_real(x.sin)
    complex_sin(Complex.from_real(x)) = Complex.from_real(x.sin)
}

/// Euler's formula in terms of the complex trigonometric functions:
/// exp(i·x) = x.cos + i·x.sin.
theorem complex_euler_formula(x: Real) {
    complex_exp(Complex.i * Complex.from_real(x)) =
        complex_cos(Complex.from_real(x)) + Complex.i * complex_sin(Complex.from_real(x))
} by {
    complex_cos_from_real(x)
    complex_cos(Complex.from_real(x)) = Complex.from_real(x.cos)
    complex_sin_from_real(x)
    complex_sin(Complex.from_real(x)) = Complex.from_real(x.sin)
    complex_euler_formula_real(x)
    complex_exp(Complex.i * Complex.from_real(x)) =
        Complex.from_real(x.cos) + Complex.i * Complex.from_real(x.sin)
    Complex.from_real(x.cos) = complex_cos(Complex.from_real(x))
    Complex.from_real(x.sin) = complex_sin(Complex.from_real(x))
    Complex.from_real(x.cos) + Complex.i * Complex.from_real(x.sin) =
        complex_cos(Complex.from_real(x)) + Complex.i * complex_sin(Complex.from_real(x))
    complex_exp(Complex.i * Complex.from_real(x)) =
        complex_cos(Complex.from_real(x)) + Complex.i * complex_sin(Complex.from_real(x))
}
