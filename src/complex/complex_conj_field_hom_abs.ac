from real import Real
from complex.complex import Complex, abs_squared_conj, abs_squared_conj_eq, from_real_one
from complex.complex_abs import modulus_conj, modulus_squared
from complex.complex_conj_hom import complex_conj_fn
from complex.complex_conj_field_hom import complex_conj_field_hom, complex_conj_field_hom_hom,
    complex_conj_field_hom_inverse
from algebra.field.field_hom import field_hom_mul
from algebra.field.field import unique_inverse

/// The complex-conjugation field homomorphism preserves squared modulus.
theorem complex_conj_field_hom_abs_squared(a: Complex) {
    complex_conj_field_hom.hom(a).abs_squared = a.abs_squared
} by {
    complex_conj_field_hom_hom
    complex_conj_fn(a) = a.conj
    abs_squared_conj_eq(a)
}

/// The complex-conjugation field homomorphism preserves complex modulus.
theorem complex_conj_field_hom_modulus(a: Complex) {
    complex_conj_field_hom.hom(a).modulus = a.modulus
} by {
    complex_conj_field_hom_hom
    complex_conj_fn(a) = a.conj
    modulus_conj(a)
}

/// Multiplying a complex number by its conjugation-field-hom image gives its squared norm.
theorem complex_mul_conj_field_hom(a: Complex) {
    a * complex_conj_field_hom.hom(a) = Complex.from_real(a.abs_squared)
} by {
    complex_conj_field_hom_hom
    complex_conj_fn(a) = a.conj
    abs_squared_conj(a)
}

/// The complex-conjugation field homomorphism preserves division by a nonzero value.
theorem complex_conj_field_hom_div(a: Complex, b: Complex) {
    b != Complex.0 implies
        complex_conj_field_hom.hom(a / b) =
        complex_conj_field_hom.hom(a) / complex_conj_field_hom.hom(b)
} by {
    if b != Complex.0 {
        field_hom_mul(complex_conj_field_hom, a, b.inverse)
        complex_conj_field_hom_inverse(b)
        complex_conj_field_hom.hom(a / b) =
            complex_conj_field_hom.hom(a) / complex_conj_field_hom.hom(b)
    }
}

/// Multiplying the conjugation-field-hom image by the original complex number gives its squared norm.
theorem complex_conj_field_hom_mul_self(a: Complex) {
    complex_conj_field_hom.hom(a) * a = Complex.from_real(a.abs_squared)
} by {
    complex_conj_field_hom_hom
    abs_squared_conj(a)
}

/// On the squared-unit circle, complex conjugation equals inverse.
theorem complex_conj_field_hom_eq_inverse_of_abs_squared_one(a: Complex) {
    a.abs_squared = Real.1 implies complex_conj_field_hom.hom(a) = a.inverse
} by {
    if a.abs_squared = Real.1 {
        complex_mul_conj_field_hom(a)
        from_real_one
        unique_inverse[Complex](a, complex_conj_field_hom.hom(a))
        complex_conj_field_hom.hom(a) = a.inverse
    }
}

/// On the unit circle, complex conjugation equals inverse.
theorem complex_conj_field_hom_eq_inverse_of_modulus_one(a: Complex) {
    a.modulus = Real.1 implies complex_conj_field_hom.hom(a) = a.inverse
} by {
    if a.modulus = Real.1 {
        modulus_squared(a)
        a.abs_squared = Real.1
        complex_conj_field_hom_eq_inverse_of_abs_squared_one(a)
        complex_conj_field_hom.hom(a) = a.inverse
    }
}
