from nat import Nat
from real import Real
from real import converges_to, converges, converges_to_unique, converges_to_imp_converges, converges_imp_converges_to, limit, tail_bound, neg_is_close, add_close, close_and_lt_imp_close
from real import prod_seq, limit_prod_seq
from real import self_close
from real import exists_small_mul_variant, mul_abs, lt_mul_pos_left
from real import abs_not_neg, lt_trans
from complex.complex import Complex, eq_by_components, neg_re, neg_im, re_mul, im_mul, complex_real_smul

/// The real-part projection of a complex sequence.
define re_seq(z: Nat -> Complex, n: Nat) -> Real {
    z(n).re
}

/// The imaginary-part projection of a complex sequence.
define im_seq(z: Nat -> Complex, n: Nat) -> Real {
    z(n).im
}

/// True if the complex sequence converges to w componentwise.
define complex_converges_to(z: Nat -> Complex, w: Complex) -> Bool {
    converges_to(re_seq(z), w.re) and converges_to(im_seq(z), w.im)
}

/// True if the complex sequence converges to some complex limit.
define complex_converges(z: Nat -> Complex) -> Bool {
    exists(w: Complex) {
        complex_converges_to(z, w)
    }
}

/// A complex sequence with a specified limit is convergent.
theorem complex_converges_to_imp_converges(z: Nat -> Complex, w: Complex) {
    complex_converges_to(z, w) implies complex_converges(z)
}

/// Componentwise complex convergence implies convergence of the real part.
theorem complex_converges_to_re(z: Nat -> Complex, w: Complex) {
    complex_converges_to(z, w) implies converges_to(re_seq(z), w.re)
}

/// Componentwise complex convergence implies convergence of the imaginary part.
theorem complex_converges_to_im(z: Nat -> Complex, w: Complex) {
    complex_converges_to(z, w) implies converges_to(im_seq(z), w.im)
}

/// Componentwise convergence of both coordinates gives complex convergence.
theorem componentwise_imp_complex_converges_to(z: Nat -> Complex, w: Complex) {
    converges_to(re_seq(z), w.re) and converges_to(im_seq(z), w.im)
    implies complex_converges_to(z, w)
}

/// A complex limit is unique when both coordinate limits are unique.
theorem complex_converges_to_unique(z: Nat -> Complex, a: Complex, b: Complex) {
    complex_converges_to(z, a) and complex_converges_to(z, b) implies a = b
} by {
    if complex_converges_to(z, a) and complex_converges_to(z, b) {
        converges_to(re_seq(z), b.re)
        converges_to(re_seq(z), a.re) and converges_to(re_seq(z), b.re)
        converges_to_unique(re_seq(z), a.re, b.re)
        converges_to(im_seq(z), b.im)
        converges_to_unique(im_seq(z), a.im, b.im)
        a.im = b.im
        eq_by_components(a, b)
        a = b
    }
}

/// The constant complex sequence with value w.
define const_seq(w: Complex, n: Nat) -> Complex {
    w
}

/// The real part of a constant complex sequence is constant.
theorem re_const_seq(w: Complex, n: Nat) {
    re_seq(const_seq(w), n) = w.re
}

/// The imaginary part of a constant complex sequence is constant.
theorem im_const_seq(w: Complex, n: Nat) {
    im_seq(const_seq(w), n) = w.im
}

/// The real part of a constant complex sequence converges to the constant value.
theorem const_re_converges_to(w: Complex) {
    converges_to(re_seq(const_seq(w)), w.re)
} by {
    forall(eps: Real) {
        if eps.is_positive {
            forall(i: Nat) {
                self_close(w.re, eps)
                re_seq(const_seq(w), i).is_close(w.re, eps)
            }
            tail_bound(re_seq(const_seq(w)), w.re, Nat.0, eps)
        }
    }
}

/// The imaginary part of a constant complex sequence converges to the constant value.
theorem const_im_converges_to(w: Complex) {
    converges_to(im_seq(const_seq(w)), w.im)
} by {
    forall(eps: Real) {
        if eps.is_positive {
            forall(i: Nat) {
                self_close(w.im, eps)
                im_seq(const_seq(w), i).is_close(w.im, eps)
            }
            tail_bound(im_seq(const_seq(w)), w.im, Nat.0, eps)
        }
    }
}

/// A constant complex sequence converges to its value.
theorem const_seq_converges_to(w: Complex) {
    complex_converges_to(const_seq(w), w)
} by {
    const_re_converges_to(w)
    const_im_converges_to(w)
}

/// A constant complex sequence is convergent.
theorem const_seq_converges(w: Complex) {
    complex_converges(const_seq(w))
} by {
    const_seq_converges_to(w)
}

/// The pointwise negation of a complex sequence.
define neg_seq(z: Nat -> Complex, n: Nat) -> Complex {
    -z(n)
}

/// The real part of the pointwise negation is the negation of the real part.
theorem re_neg_seq(z: Nat -> Complex, n: Nat) {
    re_seq(neg_seq(z), n) = -re_seq(z, n)
}

/// The imaginary part of the pointwise negation is the negation of the imaginary part.
theorem im_neg_seq(z: Nat -> Complex, n: Nat) {
    im_seq(neg_seq(z), n) = -im_seq(z, n)
}

/// The pointwise conjugate of a complex sequence.
define conj_seq(z: Nat -> Complex, n: Nat) -> Complex {
    z(n).conj
}

/// The real part is fixed by pointwise conjugation.
theorem re_conj_seq(z: Nat -> Complex, n: Nat) {
    re_seq(conj_seq(z), n) = re_seq(z, n)
}

/// The imaginary part of the pointwise conjugate is negated.
theorem im_conj_seq(z: Nat -> Complex, n: Nat) {
    im_seq(conj_seq(z), n) = -im_seq(z, n)
}

/// Helper: a sequence pointwise equal to the negation of `a` inherits convergence to `-x`.
theorem neg_pointwise_converges_to(a: Nat -> Real, b: Nat -> Real, x: Real) {
    converges_to(a, x) and forall(n: Nat) { b(n) = -a(n) }
    implies converges_to(b, -x)
} by {
    if converges_to(a, x) and forall(n: Nat) { b(n) = -a(n) } {
        converges_to(a, x) = forall(e: Real) {
            e.is_positive implies exists(m: Nat) {
                tail_bound(a, x, m, e)
            }
        }
        forall(eps: Real) {
            if eps.is_positive {
                eps.is_positive implies exists(m: Nat) {
                    tail_bound(a, x, m, eps)
                }
                exists(m: Nat) {
                    tail_bound(a, x, m, eps)
                }
                let n: Nat satisfy {
                    tail_bound(a, x, n, eps)
                }
                forall(i: Nat) {
                    if n <= i {
                        a(i).is_close(x, eps)
                        neg_is_close(a(i), x, eps)
                        b(i).is_close(-x, eps)
                    }
                }
                tail_bound(b, -x, n, eps)
            }
        }
    }
}

/// The pointwise negation of a complex sequence converges to the negation of the limit.
theorem neg_seq_converges_to(z: Nat -> Complex, w: Complex) {
    complex_converges_to(z, w) implies complex_converges_to(neg_seq(z), -w)
} by {
    if complex_converges_to(z, w) {
        converges_to(re_seq(z), w.re)
        converges_to(im_seq(z), w.im)
        forall(n: Nat) {
            re_seq(neg_seq(z), n) = -re_seq(z, n)
        }
        neg_pointwise_converges_to(re_seq(z), re_seq(neg_seq(z)), w.re)
        converges_to(re_seq(neg_seq(z)), -w.re)
        forall(n: Nat) {
            im_seq(neg_seq(z), n) = -im_seq(z, n)
        }
        neg_pointwise_converges_to(im_seq(z), im_seq(neg_seq(z)), w.im)
        converges_to(im_seq(neg_seq(z)), -w.im)
        neg_re(w)
        neg_im(w)
        complex_converges_to(neg_seq(z), -w)
    }
}

/// The pointwise negation of a convergent complex sequence is convergent.
theorem neg_seq_converges(z: Nat -> Complex) {
    complex_converges(z) implies complex_converges(neg_seq(z))
} by {
    if complex_converges(z) {
        let w: Complex satisfy {
            complex_converges_to(z, w)
        }
        neg_seq_converges_to(z, w)
    }
}

/// The pointwise conjugate of a complex sequence converges to the conjugate of the limit.
theorem conj_seq_converges_to(z: Nat -> Complex, w: Complex) {
    complex_converges_to(z, w) implies complex_converges_to(conj_seq(z), w.conj)
} by {
    if complex_converges_to(z, w) {
        converges_to(im_seq(z), w.im)
        forall(n: Nat) {
            re_seq(conj_seq(z))(n) = re_seq(z)(n)
        }
        forall(n: Nat) {
            im_seq(conj_seq(z), n) = -im_seq(z, n)
        }
        neg_pointwise_converges_to(im_seq(z), im_seq(conj_seq(z)), w.im)
        converges_to(im_seq(conj_seq(z)), -w.im)
        complex_converges_to(conj_seq(z), w.conj)
    }
}

/// The pointwise conjugate of a convergent complex sequence is convergent.
theorem conj_seq_converges(z: Nat -> Complex) {
    complex_converges(z) implies complex_converges(conj_seq(z))
} by {
    if complex_converges(z) {
        let w: Complex satisfy {
            complex_converges_to(z, w)
        }
        conj_seq_converges_to(z, w)
    }
}

/// The pointwise sum of two complex sequences.
define complex_add_seq(a: Nat -> Complex, b: Nat -> Complex, n: Nat) -> Complex {
    a(n) + b(n)
}

/// The real part of the pointwise sum is the sum of real parts.
theorem re_complex_add_seq(a: Nat -> Complex, b: Nat -> Complex, n: Nat) {
    re_seq(complex_add_seq(a, b), n) = re_seq(a, n) + re_seq(b, n)
} by {
    (a(n) + b(n)).re = a(n).re + b(n).re
}

/// The imaginary part of the pointwise sum is the sum of imaginary parts.
theorem im_complex_add_seq(a: Nat -> Complex, b: Nat -> Complex, n: Nat) {
    im_seq(complex_add_seq(a, b), n) = im_seq(a, n) + im_seq(b, n)
} by {
    (a(n) + b(n)).im = a(n).im + b(n).im
}

/// Helper: a sequence pointwise equal to a sum inherits convergence to the sum of limits.
theorem add_pointwise_converges_to(a: Nat -> Real, b: Nat -> Real, c: Nat -> Real, x: Real, y: Real) {
    converges_to(a, x) and converges_to(b, y) and forall(n: Nat) { c(n) = a(n) + b(n) }
    implies converges_to(c, x + y)
} by {
    if converges_to(a, x) and converges_to(b, y) and forall(n: Nat) { c(n) = a(n) + b(n) } {
        converges_to(a, x) = forall(e: Real) {
            e.is_positive implies exists(m: Nat) {
                tail_bound(a, x, m, e)
            }
        }
        converges_to(b, y) = forall(e: Real) {
            e.is_positive implies exists(m: Nat) {
                tail_bound(b, y, m, e)
            }
        }
        forall(eps: Real) {
            if eps.is_positive {
                let eps2: Real satisfy {
                    eps2.is_positive and eps2 + eps2 < eps
                }
                eps2.is_positive implies exists(m: Nat) {
                    tail_bound(a, x, m, eps2)
                }
                exists(m: Nat) {
                    tail_bound(a, x, m, eps2)
                }
                let n1: Nat satisfy {
                    tail_bound(a, x, n1, eps2)
                }
                eps2.is_positive implies exists(m: Nat) {
                    tail_bound(b, y, m, eps2)
                }
                exists(m: Nat) {
                    tail_bound(b, y, m, eps2)
                }
                let n2: Nat satisfy {
                    tail_bound(b, y, n2, eps2)
                }
                let n: Nat satisfy {
                    n1 <= n and n2 <= n
                }
                forall(i: Nat) {
                    if n <= i {
                        a(i).is_close(x, eps2)
                        b(i).is_close(y, eps2)
                        add_close(a(i), b(i), x, y, eps2, eps2)
                        close_and_lt_imp_close(c(i), x + y, eps2 + eps2, eps)
                        c(i).is_close(x + y, eps)
                    }
                }
                tail_bound(c, x + y, n, eps)
            }
        }
    }
}

/// The pointwise sum of two convergent complex sequences converges to the sum of limits.
theorem complex_add_seq_converges_to(a: Nat -> Complex, b: Nat -> Complex, w1: Complex, w2: Complex) {
    complex_converges_to(a, w1) and complex_converges_to(b, w2)
    implies complex_converges_to(complex_add_seq(a, b), w1 + w2)
} by {
    if complex_converges_to(a, w1) and complex_converges_to(b, w2) {
        converges_to(re_seq(a), w1.re)
        converges_to(im_seq(a), w1.im)
        converges_to(re_seq(b), w2.re)
        converges_to(im_seq(b), w2.im)
        forall(n: Nat) {
            re_seq(complex_add_seq(a, b))(n) = re_seq(a)(n) + re_seq(b)(n)
        }
        add_pointwise_converges_to(re_seq(a), re_seq(b), re_seq(complex_add_seq(a, b)), w1.re, w2.re)
        converges_to(re_seq(complex_add_seq(a, b)), w1.re + w2.re)
        forall(n: Nat) {
            im_seq(complex_add_seq(a, b))(n) = im_seq(a)(n) + im_seq(b)(n)
        }
        add_pointwise_converges_to(im_seq(a), im_seq(b), im_seq(complex_add_seq(a, b)), w1.im, w2.im)
        converges_to(im_seq(complex_add_seq(a, b)), w1.im + w2.im)
        complex_converges_to(complex_add_seq(a, b), w1 + w2)
    }
}

/// The pointwise sum of two convergent complex sequences is convergent.
theorem complex_add_seq_converges(a: Nat -> Complex, b: Nat -> Complex) {
    complex_converges(a) and complex_converges(b) implies complex_converges(complex_add_seq(a, b))
} by {
    if complex_converges(a) and complex_converges(b) {
        let w1: Complex satisfy {
            complex_converges_to(a, w1)
        }
        let w2: Complex satisfy {
            complex_converges_to(b, w2)
        }
        complex_add_seq_converges_to(a, b, w1, w2)
        exists(w: Complex) {
            complex_converges_to(complex_add_seq(a, b), w)
        }
    }
}

/// Multiplication by a fixed real preserves closeness after rescaling the tolerance.
theorem real_scalar_mul_close(c: Real, a: Real, b: Real, eps2: Real, eps: Real) {
    eps.is_positive and a.is_close(b, eps2) and c.abs * eps2 < eps
    implies (c * a).is_close(c * b, eps)
} by {
    mul_abs(c, a - b)
    if c.abs.is_positive {
        lt_mul_pos_left((a - b).abs, eps2, c.abs)
        lt_trans(c.abs * (a - b).abs, c.abs * eps2, eps)
        c.abs * (a - b).abs < eps
    } else {
        abs_not_neg(c)
        c.abs = Real.0
        c.abs * (a - b).abs = Real.0
        Real.0 < eps
        c.abs * (a - b).abs < eps
    }
    (c * a).is_close(c * b, eps)
}

/// Helper: a sequence pointwise equal to a real scalar multiple inherits convergence to the scaled limit.
theorem scalar_mul_pointwise_converges_to(a: Nat -> Real, b: Nat -> Real, c: Real, x: Real) {
    converges_to(a, x) and forall(n: Nat) { b(n) = c * a(n) }
    implies converges_to(b, c * x)
} by {
    if converges_to(a, x) and forall(n: Nat) { b(n) = c * a(n) } {
        converges_to(a, x) = forall(e: Real) {
            e.is_positive implies exists(m: Nat) {
                tail_bound(a, x, m, e)
            }
        }
        forall(eps: Real) {
            if eps.is_positive {
                exists_small_mul_variant(c, eps)
                let eps2: Real satisfy {
                    eps2.is_positive and c.abs * eps2 < eps
                }
                eps2.is_positive implies exists(m: Nat) {
                    tail_bound(a, x, m, eps2)
                }
                exists(m: Nat) {
                    tail_bound(a, x, m, eps2)
                }
                let n: Nat satisfy {
                    tail_bound(a, x, n, eps2)
                }
                forall(i: Nat) {
                    if n <= i {
                        a(i).is_close(x, eps2)
                        real_scalar_mul_close(c, a(i), x, eps2, eps)
                        b(i).is_close(c * x, eps)
                    }
                }
                tail_bound(b, c * x, n, eps)
            }
        }
    }
}

/// The pointwise complex-scalar multiple of a complex sequence.
define complex_scale_seq(c: Complex, z: Nat -> Complex, n: Nat) -> Complex {
    c * z(n)
}

/// The real part of the complex-scalar multiple follows the (ac - bd) rule.
theorem re_complex_scale_seq(c: Complex, z: Nat -> Complex, n: Nat) {
    re_seq(complex_scale_seq(c, z), n) = c.re * re_seq(z, n) - c.im * im_seq(z, n)
} by {
    re_mul(c, z(n))
}

/// The imaginary part of the complex-scalar multiple follows the (ad + bc) rule.
theorem im_complex_scale_seq(c: Complex, z: Nat -> Complex, n: Nat) {
    im_seq(complex_scale_seq(c, z), n) = c.re * im_seq(z, n) + c.im * re_seq(z, n)
} by {
    im_mul(c, z(n))
}

/// The real-part component sequence obtained by scaling the real part of z by alpha.
define re_scaled_seq(alpha: Real, z: Nat -> Complex, n: Nat) -> Real {
    alpha * re_seq(z, n)
}

/// The imaginary-part component sequence obtained by scaling the imaginary part of z by alpha.
define im_scaled_seq(alpha: Real, z: Nat -> Complex, n: Nat) -> Real {
    alpha * im_seq(z, n)
}

/// Scaling the real-part sequence preserves componentwise convergence.
theorem re_scaled_seq_converges_to(alpha: Real, z: Nat -> Complex, w: Complex) {
    complex_converges_to(z, w) implies converges_to(re_scaled_seq(alpha, z), alpha * w.re)
} by {
    if complex_converges_to(z, w) {
        converges_to(re_seq(z), w.re)
        forall(n: Nat) {
            re_scaled_seq(alpha, z)(n) = alpha * re_seq(z)(n)
        }
        scalar_mul_pointwise_converges_to(re_seq(z), re_scaled_seq(alpha, z), alpha, w.re)
        converges_to(re_scaled_seq(alpha, z), alpha * w.re)
    }
}

/// Scaling the imaginary-part sequence preserves componentwise convergence.
theorem im_scaled_seq_converges_to(alpha: Real, z: Nat -> Complex, w: Complex) {
    complex_converges_to(z, w) implies converges_to(im_scaled_seq(alpha, z), alpha * w.im)
} by {
    if complex_converges_to(z, w) {
        converges_to(im_seq(z), w.im)
        forall(n: Nat) {
            im_scaled_seq(alpha, z)(n) = alpha * im_seq(z)(n)
        }
        scalar_mul_pointwise_converges_to(im_seq(z), im_scaled_seq(alpha, z), alpha, w.im)
        converges_to(im_scaled_seq(alpha, z), alpha * w.im)
    }
}

/// The negation of a scaled imaginary-part sequence.
define neg_im_scaled_seq(alpha: Real, z: Nat -> Complex, n: Nat) -> Real {
    -im_scaled_seq(alpha, z, n)
}

/// The negation of a scaled real-part sequence.
define neg_re_scaled_seq(alpha: Real, z: Nat -> Complex, n: Nat) -> Real {
    -re_scaled_seq(alpha, z, n)
}

/// The negation of a scaled imaginary-part sequence converges to the negation of the scaled limit.
theorem neg_im_scaled_seq_converges_to(alpha: Real, z: Nat -> Complex, w: Complex) {
    complex_converges_to(z, w) implies converges_to(neg_im_scaled_seq(alpha, z), -(alpha * w.im))
} by {
    if complex_converges_to(z, w) {
        im_scaled_seq_converges_to(alpha, z, w)
        forall(n: Nat) {
            neg_im_scaled_seq(alpha, z)(n) = -im_scaled_seq(alpha, z)(n)
        }
        neg_pointwise_converges_to(im_scaled_seq(alpha, z), neg_im_scaled_seq(alpha, z), alpha * w.im)
        converges_to(neg_im_scaled_seq(alpha, z), -(alpha * w.im))
    }
}

/// The negation of a scaled real-part sequence converges to the negation of the scaled limit.
theorem neg_re_scaled_seq_converges_to(alpha: Real, z: Nat -> Complex, w: Complex) {
    complex_converges_to(z, w) implies converges_to(neg_re_scaled_seq(alpha, z), -(alpha * w.re))
} by {
    if complex_converges_to(z, w) {
        re_scaled_seq_converges_to(alpha, z, w)
        forall(n: Nat) {
            neg_re_scaled_seq(alpha, z)(n) = -re_scaled_seq(alpha, z)(n)
        }
        neg_pointwise_converges_to(re_scaled_seq(alpha, z), neg_re_scaled_seq(alpha, z), alpha * w.re)
        converges_to(neg_re_scaled_seq(alpha, z), -(alpha * w.re))
    }
}

/// The pointwise complex-scalar multiple of a convergent complex sequence converges to the scaled limit.
theorem complex_scale_seq_converges_to(c: Complex, z: Nat -> Complex, w: Complex) {
    complex_converges_to(z, w) implies complex_converges_to(complex_scale_seq(c, z), c * w)
} by {
    if complex_converges_to(z, w) {
        re_scaled_seq_converges_to(c.re, z, w)
        converges_to(re_scaled_seq(c.re, z), c.re * w.re)
        neg_im_scaled_seq_converges_to(c.im, z, w)
        converges_to(neg_im_scaled_seq(c.im, z), -(c.im * w.im))
        forall(n: Nat) {
            re_seq(complex_scale_seq(c, z), n) = c.re * re_seq(z, n) - c.im * im_seq(z, n)
            re_seq(complex_scale_seq(c, z), n) = re_scaled_seq(c.re, z, n) + neg_im_scaled_seq(c.im, z, n)
            re_seq(complex_scale_seq(c, z))(n) = re_scaled_seq(c.re, z)(n) + neg_im_scaled_seq(c.im, z)(n)
        }
        add_pointwise_converges_to(re_scaled_seq(c.re, z), neg_im_scaled_seq(c.im, z),
            re_seq(complex_scale_seq(c, z)), c.re * w.re, -(c.im * w.im))
        converges_to(re_seq(complex_scale_seq(c, z)), c.re * w.re + -(c.im * w.im))
        c.re * w.re + -(c.im * w.im) = c.re * w.re - c.im * w.im
        re_mul(c, w)

        im_scaled_seq_converges_to(c.re, z, w)
        re_scaled_seq_converges_to(c.im, z, w)
        forall(n: Nat) {
            im_seq(complex_scale_seq(c, z))(n) = im_scaled_seq(c.re, z)(n) + re_scaled_seq(c.im, z)(n)
        }
        add_pointwise_converges_to(im_scaled_seq(c.re, z), re_scaled_seq(c.im, z),
            im_seq(complex_scale_seq(c, z)), c.re * w.im, c.im * w.re)
        converges_to(im_seq(complex_scale_seq(c, z)), c.re * w.im + c.im * w.re)
        im_mul(c, w)

        complex_converges_to(complex_scale_seq(c, z), c * w)
    }
}

/// The pointwise complex-scalar multiple of a convergent complex sequence is convergent.
theorem complex_scale_seq_converges(c: Complex, z: Nat -> Complex) {
    complex_converges(z) implies complex_converges(complex_scale_seq(c, z))
} by {
    if complex_converges(z) {
        let w: Complex satisfy {
            complex_converges_to(z, w)
        }
        complex_scale_seq_converges_to(c, z, w)
        exists(v: Complex) {
            complex_converges_to(complex_scale_seq(c, z), v)
        }
    }
}

/// Adding a constant complex value to a convergent sequence shifts the limit by that constant.
theorem complex_add_const_converges_to(z: Nat -> Complex, w: Complex, c: Complex) {
    complex_converges_to(z, w) implies complex_converges_to(complex_add_seq(z, const_seq(c)), w + c)
} by {
    if complex_converges_to(z, w) {
        const_seq_converges_to(c)
        complex_add_seq_converges_to(z, const_seq(c), w, c)
    }
}

/// Adding a constant complex value to a convergent sequence yields a convergent sequence.
theorem complex_add_const_converges(z: Nat -> Complex, c: Complex) {
    complex_converges(z) implies complex_converges(complex_add_seq(z, const_seq(c)))
} by {
    if complex_converges(z) {
        let w: Complex satisfy {
            complex_converges_to(z, w)
        }
        complex_add_const_converges_to(z, w, c)
        complex_converges_to(complex_add_seq(z, const_seq(c)), w + c)
    }
}

/// The pointwise difference of two complex sequences.
define complex_sub_seq(a: Nat -> Complex, b: Nat -> Complex, n: Nat) -> Complex {
    a(n) - b(n)
}

/// Pointwise subtraction equals pointwise addition with the second sequence negated.
theorem complex_sub_seq_eq_add_neg(a: Nat -> Complex, b: Nat -> Complex) {
    complex_sub_seq(a, b) = complex_add_seq(a, neg_seq(b))
} by {
    forall(n: Nat) {
        complex_add_seq(a, neg_seq(b), n) = a(n) - b(n)
        complex_sub_seq(a, b)(n) = complex_add_seq(a, neg_seq(b))(n)
    }
}

/// The pointwise difference of two convergent complex sequences converges to the difference of limits.
theorem complex_sub_seq_converges_to(a: Nat -> Complex, b: Nat -> Complex, w1: Complex, w2: Complex) {
    complex_converges_to(a, w1) and complex_converges_to(b, w2)
    implies complex_converges_to(complex_sub_seq(a, b), w1 - w2)
} by {
    if complex_converges_to(a, w1) and complex_converges_to(b, w2) {
        neg_seq_converges_to(b, w2)
        complex_add_seq_converges_to(a, neg_seq(b), w1, -w2)
        complex_sub_seq_eq_add_neg(a, b)
        complex_converges_to(complex_sub_seq(a, b), w1 - w2)
    }
}

/// Subtracting a constant complex value from a convergent sequence shifts the limit by that constant.
theorem complex_sub_const_converges_to(z: Nat -> Complex, w: Complex, c: Complex) {
    complex_converges_to(z, w) implies complex_converges_to(complex_sub_seq(z, const_seq(c)), w - c)
} by {
    if complex_converges_to(z, w) {
        const_seq_converges_to(c)
        complex_sub_seq_converges_to(z, const_seq(c), w, c)
    }
}

/// Subtracting a constant complex value from a convergent sequence yields a convergent sequence.
theorem complex_sub_const_converges(z: Nat -> Complex, c: Complex) {
    complex_converges(z) implies complex_converges(complex_sub_seq(z, const_seq(c)))
} by {
    if complex_converges(z) {
        let w: Complex satisfy {
            complex_converges_to(z, w)
        }
        complex_sub_const_converges_to(z, w, c)
        complex_converges_to(complex_sub_seq(z, const_seq(c)), w - c)
    }
}

/// Helper: a sequence pointwise equal to a product inherits convergence to the product of limits.
theorem prod_pointwise_converges_to(a: Nat -> Real, b: Nat -> Real, c: Nat -> Real, x: Real, y: Real) {
    converges_to(a, x) and converges_to(b, y) and forall(n: Nat) { c(n) = a(n) * b(n) }
    implies converges_to(c, x * y)
} by {
    if converges_to(a, x) and converges_to(b, y) and forall(n: Nat) { c(n) = a(n) * b(n) } {
        converges_to_imp_converges(a, x)
        converges_to_imp_converges(b, y)
        converges_to_unique(a, limit(a), x)
        limit(a) = x
        converges_to_unique(b, limit(b), y)
        limit(b) = y
        limit_prod_seq(a, b)
        converges_to(prod_seq(a, b), limit(a) * limit(b))
        forall(n: Nat) {
            prod_seq(a, b)(n) = c(n)
        }
        converges_to(c, x * y)
    }
}

/// The pointwise product of two complex sequences.
define complex_mul_seq(a: Nat -> Complex, b: Nat -> Complex, n: Nat) -> Complex {
    a(n) * b(n)
}

/// The real part of the pointwise complex product follows the (ac - bd) rule.
theorem re_complex_mul_seq(a: Nat -> Complex, b: Nat -> Complex, n: Nat) {
    re_seq(complex_mul_seq(a, b), n) = re_seq(a, n) * re_seq(b, n) - im_seq(a, n) * im_seq(b, n)
} by {
    re_mul(a(n), b(n))
}

/// The imaginary part of the pointwise complex product follows the (ad + bc) rule.
theorem im_complex_mul_seq(a: Nat -> Complex, b: Nat -> Complex, n: Nat) {
    im_seq(complex_mul_seq(a, b), n) = re_seq(a, n) * im_seq(b, n) + im_seq(a, n) * re_seq(b, n)
} by {
    im_mul(a(n), b(n))
}

/// Multiplying a sequence pointwise by a constant equals the complex-scalar action.
theorem complex_mul_const_seq_eq_scale(z: Nat -> Complex, c: Complex) {
    complex_mul_seq(const_seq(c), z) = complex_scale_seq(c, z)
} by {
    forall(n: Nat) {
        complex_mul_seq(const_seq(c), z)(n) = complex_scale_seq(c, z)(n)
    }
}

/// Multiplying a convergent sequence by a constant on the left converges to the scaled limit.
theorem complex_mul_const_left_converges_to(z: Nat -> Complex, w: Complex, c: Complex) {
    complex_converges_to(z, w) implies complex_converges_to(complex_mul_seq(const_seq(c), z), c * w)
} by {
    if complex_converges_to(z, w) {
        complex_scale_seq_converges_to(c, z, w)
        complex_mul_const_seq_eq_scale(z, c)
        complex_converges_to(complex_mul_seq(const_seq(c), z), c * w)
    }
}

/// The pointwise real-scalar multiple of a complex sequence.
define complex_real_smul_seq(alpha: Real, z: Nat -> Complex, n: Nat) -> Complex {
    complex_real_smul(alpha, z(n))
}

/// The real part of the real-scalar multiple is the scalar times the real part.
theorem re_complex_real_smul_seq(alpha: Real, z: Nat -> Complex, n: Nat) {
    re_seq(complex_real_smul_seq(alpha, z), n) = alpha * re_seq(z, n)
} by {
    complex_real_smul(alpha, z(n)) = Complex.new(alpha * z(n).re, alpha * z(n).im)
}

/// The imaginary part of the real-scalar multiple is the scalar times the imaginary part.
theorem im_complex_real_smul_seq(alpha: Real, z: Nat -> Complex, n: Nat) {
    im_seq(complex_real_smul_seq(alpha, z), n) = alpha * im_seq(z, n)
} by {
    complex_real_smul(alpha, z(n)) = Complex.new(alpha * z(n).re, alpha * z(n).im)
}

/// The pointwise real-scalar multiple of a convergent complex sequence converges to the scaled limit.
theorem complex_real_smul_seq_converges_to(alpha: Real, z: Nat -> Complex, w: Complex) {
    complex_converges_to(z, w)
    implies complex_converges_to(complex_real_smul_seq(alpha, z), complex_real_smul(alpha, w))
} by {
    if complex_converges_to(z, w) {
        converges_to(re_seq(z), w.re)
        converges_to(im_seq(z), w.im)
        forall(n: Nat) {
            re_seq(complex_real_smul_seq(alpha, z))(n) = alpha * re_seq(z)(n)
        }
        scalar_mul_pointwise_converges_to(re_seq(z), re_seq(complex_real_smul_seq(alpha, z)), alpha, w.re)
        converges_to(re_seq(complex_real_smul_seq(alpha, z)), alpha * w.re)
        forall(n: Nat) {
            im_seq(complex_real_smul_seq(alpha, z))(n) = alpha * im_seq(z)(n)
        }
        scalar_mul_pointwise_converges_to(im_seq(z), im_seq(complex_real_smul_seq(alpha, z)), alpha, w.im)
        converges_to(im_seq(complex_real_smul_seq(alpha, z)), alpha * w.im)
        complex_converges_to(complex_real_smul_seq(alpha, z), complex_real_smul(alpha, w))
    }
}

/// The pointwise real-scalar multiple of a convergent complex sequence is convergent.
theorem complex_real_smul_seq_converges(alpha: Real, z: Nat -> Complex) {
    complex_converges(z) implies complex_converges(complex_real_smul_seq(alpha, z))
} by {
    if complex_converges(z) {
        let w: Complex satisfy {
            complex_converges_to(z, w)
        }
        complex_real_smul_seq_converges_to(alpha, z, w)
        exists(v: Complex) {
            complex_converges_to(complex_real_smul_seq(alpha, z), v)
        }
    }
}

/// The pointwise product of real-part and imaginary-part sequences of two complex sequences.
define re_re_prod_seq(a: Nat -> Complex, b: Nat -> Complex, n: Nat) -> Real {
    re_seq(a, n) * re_seq(b, n)
}

/// The pointwise product of imaginary-part sequences of two complex sequences.
define im_im_prod_seq(a: Nat -> Complex, b: Nat -> Complex, n: Nat) -> Real {
    im_seq(a, n) * im_seq(b, n)
}

/// The pointwise product of the real-part of a and imaginary-part of b.
define re_im_prod_seq(a: Nat -> Complex, b: Nat -> Complex, n: Nat) -> Real {
    re_seq(a, n) * im_seq(b, n)
}

/// The pointwise product of the imaginary-part of a and real-part of b.
define im_re_prod_seq(a: Nat -> Complex, b: Nat -> Complex, n: Nat) -> Real {
    im_seq(a, n) * re_seq(b, n)
}

/// The negation of `im_im_prod_seq`.
define neg_im_im_prod_seq(a: Nat -> Complex, b: Nat -> Complex, n: Nat) -> Real {
    -im_im_prod_seq(a, b, n)
}

/// The pointwise product of two convergent complex sequences converges to the product of limits.
theorem complex_mul_seq_converges_to(a: Nat -> Complex, b: Nat -> Complex, w1: Complex, w2: Complex) {
    complex_converges_to(a, w1) and complex_converges_to(b, w2)
    implies complex_converges_to(complex_mul_seq(a, b), w1 * w2)
} by {
    if complex_converges_to(a, w1) and complex_converges_to(b, w2) {
        converges_to(re_seq(a), w1.re)
        converges_to(im_seq(a), w1.im)
        converges_to(re_seq(b), w2.re)
        converges_to(im_seq(b), w2.im)

        // re_re_prod_seq converges to w1.re * w2.re
        forall(n: Nat) {
            re_re_prod_seq(a, b)(n) = re_seq(a)(n) * re_seq(b)(n)
        }
        prod_pointwise_converges_to(re_seq(a), re_seq(b), re_re_prod_seq(a, b), w1.re, w2.re)
        converges_to(re_re_prod_seq(a, b), w1.re * w2.re)

        // im_im_prod_seq converges to w1.im * w2.im
        forall(n: Nat) {
            im_im_prod_seq(a, b)(n) = im_seq(a)(n) * im_seq(b)(n)
        }
        prod_pointwise_converges_to(im_seq(a), im_seq(b), im_im_prod_seq(a, b), w1.im, w2.im)
        converges_to(im_im_prod_seq(a, b), w1.im * w2.im)

        // neg_im_im_prod_seq converges to -(w1.im * w2.im)
        forall(n: Nat) {
            neg_im_im_prod_seq(a, b)(n) = -im_im_prod_seq(a, b)(n)
        }
        neg_pointwise_converges_to(im_im_prod_seq(a, b), neg_im_im_prod_seq(a, b), w1.im * w2.im)
        converges_to(neg_im_im_prod_seq(a, b), -(w1.im * w2.im))

        // re_seq(complex_mul_seq(a, b)) = re_re_prod_seq + neg_im_im_prod_seq
        forall(n: Nat) {
            re_seq(complex_mul_seq(a, b), n) = re_seq(a, n) * re_seq(b, n) - im_seq(a, n) * im_seq(b, n)
            re_seq(a, n) * re_seq(b, n) - im_seq(a, n) * im_seq(b, n) = re_re_prod_seq(a, b, n) + neg_im_im_prod_seq(a, b, n)
            re_seq(complex_mul_seq(a, b))(n) = re_re_prod_seq(a, b)(n) + neg_im_im_prod_seq(a, b)(n)
        }
        add_pointwise_converges_to(re_re_prod_seq(a, b), neg_im_im_prod_seq(a, b),
            re_seq(complex_mul_seq(a, b)), w1.re * w2.re, -(w1.im * w2.im))
        converges_to(re_seq(complex_mul_seq(a, b)), w1.re * w2.re + -(w1.im * w2.im))
        w1.re * w2.re + -(w1.im * w2.im) = w1.re * w2.re - w1.im * w2.im
        re_mul(w1, w2)

        // re_im_prod_seq converges to w1.re * w2.im
        forall(n: Nat) {
            re_im_prod_seq(a, b)(n) = re_seq(a)(n) * im_seq(b)(n)
        }
        prod_pointwise_converges_to(re_seq(a), im_seq(b), re_im_prod_seq(a, b), w1.re, w2.im)
        converges_to(re_im_prod_seq(a, b), w1.re * w2.im)

        // im_re_prod_seq converges to w1.im * w2.re
        forall(n: Nat) {
            im_re_prod_seq(a, b)(n) = im_seq(a)(n) * re_seq(b)(n)
        }
        prod_pointwise_converges_to(im_seq(a), re_seq(b), im_re_prod_seq(a, b), w1.im, w2.re)
        converges_to(im_re_prod_seq(a, b), w1.im * w2.re)

        // im_seq(complex_mul_seq(a, b)) = re_im_prod_seq + im_re_prod_seq
        forall(n: Nat) {
            im_seq(complex_mul_seq(a, b))(n) = re_im_prod_seq(a, b)(n) + im_re_prod_seq(a, b)(n)
        }
        add_pointwise_converges_to(re_im_prod_seq(a, b), im_re_prod_seq(a, b),
            im_seq(complex_mul_seq(a, b)), w1.re * w2.im, w1.im * w2.re)
        converges_to(im_seq(complex_mul_seq(a, b)), w1.re * w2.im + w1.im * w2.re)
        im_mul(w1, w2)

        complex_converges_to(complex_mul_seq(a, b), w1 * w2)
    }
}

/// The pointwise product of two convergent complex sequences is convergent.
theorem complex_mul_seq_converges(a: Nat -> Complex, b: Nat -> Complex) {
    complex_converges(a) and complex_converges(b) implies complex_converges(complex_mul_seq(a, b))
} by {
    if complex_converges(a) and complex_converges(b) {
        let w1: Complex satisfy {
            complex_converges_to(a, w1)
        }
        let w2: Complex satisfy {
            complex_converges_to(b, w2)
        }
        complex_mul_seq_converges_to(a, b, w1, w2)
        exists(w: Complex) {
            complex_converges_to(complex_mul_seq(a, b), w)
        }
    }
}

/// Multiplying a sequence pointwise on the right by a constant equals the complex-scalar action.
theorem complex_mul_const_right_seq_eq_scale(z: Nat -> Complex, c: Complex) {
    complex_mul_seq(z, const_seq(c)) = complex_scale_seq(c, z)
} by {
    forall(n: Nat) {
        complex_mul_seq(z, const_seq(c), n) = c * z(n)
        complex_mul_seq(z, const_seq(c))(n) = complex_scale_seq(c, z)(n)
    }
}

/// Multiplying a convergent sequence by a constant on the right converges to the scaled limit.
theorem complex_mul_const_right_converges_to(z: Nat -> Complex, w: Complex, c: Complex) {
    complex_converges_to(z, w) implies complex_converges_to(complex_mul_seq(z, const_seq(c)), c * w)
} by {
    if complex_converges_to(z, w) {
        complex_scale_seq_converges_to(c, z, w)
        complex_mul_const_right_seq_eq_scale(z, c)
        complex_converges_to(complex_mul_seq(z, const_seq(c)), c * w)
    }
}

/// Multiplying a convergent sequence by a constant on the left yields a convergent sequence.
theorem complex_mul_const_left_converges(z: Nat -> Complex, c: Complex) {
    complex_converges(z) implies complex_converges(complex_mul_seq(const_seq(c), z))
} by {
    if complex_converges(z) {
        let w: Complex satisfy {
            complex_converges_to(z, w)
        }
        complex_mul_const_left_converges_to(z, w, c)
        complex_converges_to(complex_mul_seq(const_seq(c), z), c * w)
    }
}

/// Multiplying a convergent sequence by a constant on the right yields a convergent sequence.
theorem complex_mul_const_right_converges(z: Nat -> Complex, c: Complex) {
    complex_converges(z) implies complex_converges(complex_mul_seq(z, const_seq(c)))
} by {
    if complex_converges(z) {
        let w: Complex satisfy {
            complex_converges_to(z, w)
        }
        complex_mul_const_right_converges_to(z, w, c)
        complex_converges_to(complex_mul_seq(z, const_seq(c)), c * w)
    }
}

/// The pointwise difference of two convergent complex sequences is convergent.
theorem complex_sub_seq_converges(a: Nat -> Complex, b: Nat -> Complex) {
    complex_converges(a) and complex_converges(b) implies complex_converges(complex_sub_seq(a, b))
} by {
    if complex_converges(a) and complex_converges(b) {
        let w1: Complex satisfy {
            complex_converges_to(a, w1)
        }
        let w2: Complex satisfy {
            complex_converges_to(b, w2)
        }
        complex_sub_seq_converges_to(a, b, w1, w2)
        exists(w: Complex) {
            complex_converges_to(complex_sub_seq(a, b), w)
        }
    }
}

/// The limit of a complex sequence: the value to which it converges if it converges,
/// and zero otherwise.
let complex_limit(z: Nat -> Complex) -> lim: Complex satisfy {
    (complex_converges(z) implies complex_converges_to(z, lim)) and
    (not complex_converges(z) implies lim = Complex.new(Real.0, Real.0))
} by {
    if complex_converges(z) {
        let w: Complex satisfy {
            complex_converges_to(z, w)
        }
        exists(v: Complex) {
            (complex_converges(z) implies complex_converges_to(z, v)) and
            (not complex_converges(z) implies v = Complex.new(Real.0, Real.0))
        }
    } else {
        let w: Complex = Complex.new(Real.0, Real.0)
        exists(v: Complex) {
            (complex_converges(z) implies complex_converges_to(z, v)) and
            (not complex_converges(z) implies v = Complex.new(Real.0, Real.0))
        }
    }
}

/// A convergent complex sequence converges to its limit.
theorem complex_converges_imp_converges_to_limit(z: Nat -> Complex) {
    complex_converges(z) implies complex_converges_to(z, complex_limit(z))
} by {
    if complex_converges(z) {
        complex_converges_to(z, complex_limit(z))
    }
}

/// If a complex sequence converges to w, then its limit equals w.
theorem complex_converges_to_imp_limit(z: Nat -> Complex, w: Complex) {
    complex_converges_to(z, w) implies complex_limit(z) = w
} by {
    if complex_converges_to(z, w) {
        complex_converges_to_imp_converges(z, w)
        complex_converges_to(z, complex_limit(z))
        complex_converges_to_unique(z, complex_limit(z), w)
    }
}

/// The limit of a constant complex sequence is its value.
theorem complex_limit_of_const_seq(w: Complex) {
    complex_limit(const_seq(w)) = w
} by {
    const_seq_converges_to(w)
    complex_converges_to_imp_limit(const_seq(w), w)
}

/// The real part of the complex limit equals the limit of the real-part sequence.
theorem complex_limit_re(z: Nat -> Complex) {
    complex_converges(z) implies limit(re_seq(z)) = complex_limit(z).re
} by {
    if complex_converges(z) {
        converges_to(re_seq(z), complex_limit(z).re)
        converges_to_imp_converges(re_seq(z), complex_limit(z).re)
        converges_imp_converges_to(re_seq(z))
        converges_to_unique(re_seq(z), limit(re_seq(z)), complex_limit(z).re)
        limit(re_seq(z)) = complex_limit(z).re
    }
}

/// The imaginary part of the complex limit equals the limit of the imaginary-part sequence.
theorem complex_limit_im(z: Nat -> Complex) {
    complex_converges(z) implies limit(im_seq(z)) = complex_limit(z).im
} by {
    if complex_converges(z) {
        converges_to(im_seq(z), complex_limit(z).im)
        converges_to_imp_converges(im_seq(z), complex_limit(z).im)
        converges_imp_converges_to(im_seq(z))
        converges_to_unique(im_seq(z), limit(im_seq(z)), complex_limit(z).im)
        limit(im_seq(z)) = complex_limit(z).im
    }
}

/// The limit of the negation is the negation of the limit.
theorem complex_limit_of_neg_seq(z: Nat -> Complex) {
    complex_converges(z) implies complex_limit(neg_seq(z)) = -complex_limit(z)
} by {
    if complex_converges(z) {
        neg_seq_converges_to(z, complex_limit(z))
        complex_converges_to(neg_seq(z), -complex_limit(z))
        complex_converges_to_imp_limit(neg_seq(z), -complex_limit(z))
    }
}

/// The limit of the conjugate is the conjugate of the limit.
theorem complex_limit_of_conj_seq(z: Nat -> Complex) {
    complex_converges(z) implies complex_limit(conj_seq(z)) = complex_limit(z).conj
} by {
    if complex_converges(z) {
        conj_seq_converges_to(z, complex_limit(z))
        complex_converges_to(conj_seq(z), complex_limit(z).conj)
        complex_converges_to_imp_limit(conj_seq(z), complex_limit(z).conj)
    }
}

/// The limit of a scalar multiple is the scalar multiple of the limit.
theorem complex_limit_of_scale_seq(c: Complex, z: Nat -> Complex) {
    complex_converges(z) implies complex_limit(complex_scale_seq(c, z)) = c * complex_limit(z)
} by {
    if complex_converges(z) {
        complex_scale_seq_converges_to(c, z, complex_limit(z))
        complex_converges_to(complex_scale_seq(c, z), c * complex_limit(z))
        complex_converges_to_imp_limit(complex_scale_seq(c, z), c * complex_limit(z))
    }
}

/// The limit of the pointwise sum of two convergent complex sequences is the sum of the limits.
theorem complex_limit_of_add_seq(a: Nat -> Complex, b: Nat -> Complex) {
    complex_converges(a) and complex_converges(b)
    implies complex_limit(complex_add_seq(a, b)) = complex_limit(a) + complex_limit(b)
} by {
    if complex_converges(a) and complex_converges(b) {
        complex_converges_imp_converges_to_limit(a)
        complex_converges_imp_converges_to_limit(b)
        complex_add_seq_converges_to(a, b, complex_limit(a), complex_limit(b))
        complex_converges_to_imp_limit(complex_add_seq(a, b), complex_limit(a) + complex_limit(b))
        complex_limit(complex_add_seq(a, b)) = complex_limit(a) + complex_limit(b)
    }
}

/// The limit of the pointwise difference of two convergent complex sequences is the difference of the limits.
theorem complex_limit_of_sub_seq(a: Nat -> Complex, b: Nat -> Complex) {
    complex_converges(a) and complex_converges(b)
    implies complex_limit(complex_sub_seq(a, b)) = complex_limit(a) - complex_limit(b)
} by {
    if complex_converges(a) and complex_converges(b) {
        complex_converges_imp_converges_to_limit(a)
        complex_converges_imp_converges_to_limit(b)
        complex_sub_seq_converges_to(a, b, complex_limit(a), complex_limit(b))
        complex_converges_to_imp_limit(complex_sub_seq(a, b), complex_limit(a) - complex_limit(b))
        complex_limit(complex_sub_seq(a, b)) = complex_limit(a) - complex_limit(b)
    }
}

/// The limit of a real-scalar multiple is the real-scalar multiple of the limit.
theorem complex_limit_of_real_smul_seq(alpha: Real, z: Nat -> Complex) {
    complex_converges(z)
    implies complex_limit(complex_real_smul_seq(alpha, z)) = complex_real_smul(alpha, complex_limit(z))
} by {
    if complex_converges(z) {
        complex_real_smul_seq_converges_to(alpha, z, complex_limit(z))
        complex_converges_to(complex_real_smul_seq(alpha, z), complex_real_smul(alpha, complex_limit(z)))
        complex_converges_to_imp_limit(complex_real_smul_seq(alpha, z), complex_real_smul(alpha, complex_limit(z)))
    }
}

/// The limit of a sequence shifted by a constant is the limit shifted by that constant.
theorem complex_limit_of_add_const(z: Nat -> Complex, c: Complex) {
    complex_converges(z)
    implies complex_limit(complex_add_seq(z, const_seq(c))) = complex_limit(z) + c
} by {
    if complex_converges(z) {
        complex_add_const_converges_to(z, complex_limit(z), c)
        complex_converges_to(complex_add_seq(z, const_seq(c)), complex_limit(z) + c)
        complex_converges_to_imp_limit(complex_add_seq(z, const_seq(c)), complex_limit(z) + c)
    }
}

/// The limit of a sequence with a constant subtracted is the limit with that constant subtracted.
theorem complex_limit_of_sub_const(z: Nat -> Complex, c: Complex) {
    complex_converges(z)
    implies complex_limit(complex_sub_seq(z, const_seq(c))) = complex_limit(z) - c
} by {
    if complex_converges(z) {
        complex_sub_const_converges_to(z, complex_limit(z), c)
        complex_converges_to(complex_sub_seq(z, const_seq(c)), complex_limit(z) - c)
        complex_converges_to_imp_limit(complex_sub_seq(z, const_seq(c)), complex_limit(z) - c)
    }
}

/// The limit of the pointwise product of two convergent complex sequences is the product of the limits.
theorem complex_limit_of_mul_seq(a: Nat -> Complex, b: Nat -> Complex) {
    complex_converges(a) and complex_converges(b)
    implies complex_limit(complex_mul_seq(a, b)) = complex_limit(a) * complex_limit(b)
} by {
    if complex_converges(a) and complex_converges(b) {
        complex_converges_imp_converges_to_limit(a)
        complex_converges_imp_converges_to_limit(b)
        complex_mul_seq_converges_to(a, b, complex_limit(a), complex_limit(b))
        complex_converges_to_imp_limit(complex_mul_seq(a, b), complex_limit(a) * complex_limit(b))
        complex_limit(complex_mul_seq(a, b)) = complex_limit(a) * complex_limit(b)
    }
}


/// The limit of multiplying a convergent sequence by a constant on the left is the scaled limit.
theorem complex_limit_of_mul_const_left(z: Nat -> Complex, c: Complex) {
    complex_converges(z)
    implies complex_limit(complex_mul_seq(const_seq(c), z)) = c * complex_limit(z)
} by {
    if complex_converges(z) {
        complex_mul_const_left_converges_to(z, complex_limit(z), c)
        complex_converges_to(complex_mul_seq(const_seq(c), z), c * complex_limit(z))
        complex_converges_to_imp_limit(complex_mul_seq(const_seq(c), z), c * complex_limit(z))
    }
}

/// The limit of multiplying a convergent sequence by a constant on the right is the scaled limit.
theorem complex_limit_of_mul_const_right(z: Nat -> Complex, c: Complex) {
    complex_converges(z)
    implies complex_limit(complex_mul_seq(z, const_seq(c))) = c * complex_limit(z)
} by {
    if complex_converges(z) {
        complex_mul_const_right_converges_to(z, complex_limit(z), c)
        complex_converges_to(complex_mul_seq(z, const_seq(c)), c * complex_limit(z))
        complex_converges_to_imp_limit(complex_mul_seq(z, const_seq(c)), c * complex_limit(z))
    }
}

/// Adding a constant complex value to a convergent sequence on the left shifts the limit by that constant.
theorem complex_add_const_left_converges_to(c: Complex, z: Nat -> Complex, w: Complex) {
    complex_converges_to(z, w) implies complex_converges_to(complex_add_seq(const_seq(c), z), c + w)
} by {
    if complex_converges_to(z, w) {
        const_seq_converges_to(c)
        complex_add_seq_converges_to(const_seq(c), z, c, w)
    }
}

/// Adding a constant complex value to a convergent sequence on the left yields a convergent sequence.
theorem complex_add_const_left_converges(c: Complex, z: Nat -> Complex) {
    complex_converges(z) implies complex_converges(complex_add_seq(const_seq(c), z))
} by {
    if complex_converges(z) {
        let w: Complex satisfy {
            complex_converges_to(z, w)
        }
        complex_add_const_left_converges_to(c, z, w)
        complex_converges_to(complex_add_seq(const_seq(c), z), c + w)
    }
}

/// Subtracting a convergent sequence from a constant complex value yields a convergent sequence whose limit
/// is that constant minus the limit.
theorem complex_const_sub_converges_to(c: Complex, z: Nat -> Complex, w: Complex) {
    complex_converges_to(z, w) implies complex_converges_to(complex_sub_seq(const_seq(c), z), c - w)
} by {
    if complex_converges_to(z, w) {
        const_seq_converges_to(c)
        complex_sub_seq_converges_to(const_seq(c), z, c, w)
    }
}

/// Subtracting a convergent sequence from a constant complex value yields a convergent sequence.
theorem complex_const_sub_converges(c: Complex, z: Nat -> Complex) {
    complex_converges(z) implies complex_converges(complex_sub_seq(const_seq(c), z))
} by {
    if complex_converges(z) {
        let w: Complex satisfy {
            complex_converges_to(z, w)
        }
        complex_const_sub_converges_to(c, z, w)
        complex_converges_to(complex_sub_seq(const_seq(c), z), c - w)
    }
}

/// The limit of a constant complex value added to a convergent sequence on the left is that constant plus the limit.
theorem complex_limit_of_add_const_left(c: Complex, z: Nat -> Complex) {
    complex_converges(z)
    implies complex_limit(complex_add_seq(const_seq(c), z)) = c + complex_limit(z)
} by {
    if complex_converges(z) {
        complex_add_const_left_converges_to(c, z, complex_limit(z))
        complex_converges_to(complex_add_seq(const_seq(c), z), c + complex_limit(z))
        complex_converges_to_imp_limit(complex_add_seq(const_seq(c), z), c + complex_limit(z))
    }
}

/// The limit of a constant complex value minus a convergent sequence is that constant minus the limit.
theorem complex_limit_of_const_sub(c: Complex, z: Nat -> Complex) {
    complex_converges(z)
    implies complex_limit(complex_sub_seq(const_seq(c), z)) = c - complex_limit(z)
} by {
    if complex_converges(z) {
        complex_const_sub_converges_to(c, z, complex_limit(z))
        complex_converges_to(complex_sub_seq(const_seq(c), z), c - complex_limit(z))
        complex_converges_to_imp_limit(complex_sub_seq(const_seq(c), z), c - complex_limit(z))
    }
}
