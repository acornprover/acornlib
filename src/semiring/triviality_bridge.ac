from data.basic.functions import function_extensionality
from semiring.base import Semiring, trivial_semiring

/// In a semiring with `0 = 1`, every element is equal to zero.
theorem semiring_zero_eq_one_imp_all_eq_zero[S: Semiring](x: S) {
    S.0 = S.1 implies x = S.0
} by {
    if S.0 = S.1 {
        trivial_semiring[S]
        x = S.0
    }
}

/// In a semiring with `0 = 1`, every element is equal to one.
theorem semiring_zero_eq_one_imp_all_eq_one[S: Semiring](x: S) {
    S.0 = S.1 implies x = S.1
} by {
    if S.0 = S.1 {
        semiring_zero_eq_one_imp_all_eq_zero[S](x)
        x = S.0
        x = S.1
    }
}

/// In a semiring with `0 = 1`, any two elements are equal.
theorem semiring_zero_eq_one_imp_eq[S: Semiring](x: S, y: S) {
    S.0 = S.1 implies x = y
} by {
    if S.0 = S.1 {
        semiring_zero_eq_one_imp_all_eq_zero[S](x)
        semiring_zero_eq_one_imp_all_eq_zero[S](y)
        x = S.0
        y = S.0
        x = y
    }
}

/// If a semiring has two unequal elements, then its zero and one are distinct.
theorem semiring_nontrivial_of_exists_ne[S: Semiring](x: S, y: S) {
    x != y implies S.0 != S.1
} by {
    if x != y {
        if S.0 = S.1 {
            semiring_zero_eq_one_imp_eq[S](x, y)
            x = y
            false
        }
        S.0 != S.1
    }
}

/// In a semiring with `0 = 1`, all functions into the semiring are extensionally equal.
theorem semiring_zero_eq_one_imp_function_ext[A, S: Semiring](f: A -> S, g: A -> S) {
    S.0 = S.1 implies f = g
} by {
    if S.0 = S.1 {
        forall(a: A) {
            semiring_zero_eq_one_imp_eq[S](f(a), g(a))
            f(a) = g(a)
        }
        function_extensionality(f, g)
        f = g
    }
}
