from data.basic.logic import iff_bool_imp_eq
from semiring.base import Semiring
from semiring.triviality_bridge import semiring_zero_eq_one_imp_all_eq_zero,
    semiring_zero_eq_one_imp_all_eq_one, semiring_zero_eq_one_imp_eq,
    semiring_nontrivial_of_exists_ne

/// A semiring has equal zero and one exactly when every element is zero.
theorem semiring_zero_eq_one_iff_all_eq_zero[S: Semiring] {
    (S.0 = S.1) = (forall(x: S) { x = S.0 })
} by {
    if S.0 = S.1 {
        forall(x: S) {
            semiring_zero_eq_one_imp_all_eq_zero[S](x)
            x = S.0
        }
    }
    if forall(x: S) { x = S.0 } {
        S.1 = S.0
        S.0 = S.1
    }
    (S.0 = S.1 implies forall(x: S) { x = S.0 })
    (forall(x: S) { x = S.0 } implies S.0 = S.1)
    iff_bool_imp_eq(S.0 = S.1, forall(x: S) { x = S.0 })
}

/// A semiring has equal zero and one exactly when every element is one.
theorem semiring_zero_eq_one_iff_all_eq_one[S: Semiring] {
    (S.0 = S.1) = (forall(x: S) { x = S.1 })
} by {
    if S.0 = S.1 {
        forall(x: S) {
            semiring_zero_eq_one_imp_all_eq_one[S](x)
            x = S.1
        }
    }
    if forall(x: S) { x = S.1 } {
        S.0 = S.1
    }
    (S.0 = S.1 implies forall(x: S) { x = S.1 })
    (forall(x: S) { x = S.1 } implies S.0 = S.1)
    iff_bool_imp_eq(S.0 = S.1, forall(x: S) { x = S.1 })
}

/// A semiring has equal zero and one exactly when all elements are equal.
theorem semiring_zero_eq_one_iff_all_eq[S: Semiring] {
    (S.0 = S.1) = (forall(x: S, y: S) { x = y })
} by {
    if S.0 = S.1 {
        forall(x: S, y: S) {
            semiring_zero_eq_one_imp_eq[S](x, y)
            x = y
        }
    }
    if forall(x: S, y: S) { x = y } {
        S.0 = S.1
    }
    (S.0 = S.1 implies forall(x: S, y: S) { x = y })
    (forall(x: S, y: S) { x = y } implies S.0 = S.1)
    iff_bool_imp_eq(S.0 = S.1, forall(x: S, y: S) { x = y })
}

/// A semiring has distinct zero and one exactly when it has two distinct elements.
theorem semiring_nontrivial_iff_exists_ne[S: Semiring] {
    (S.0 != S.1) = (exists(x: S, y: S) { x != y })
} by {
    if S.0 != S.1 {
        exists(x: S, y: S) {
            x = S.0 and y = S.1 and x != y
        }
    }
    if exists(x: S, y: S) { x != y } {
        let x: S satisfy {
            exists(y: S) {
                x != y
            }
        }
        let y: S satisfy {
            x != y
        }
        semiring_nontrivial_of_exists_ne[S](x, y)
        S.0 != S.1
    }
    (S.0 != S.1 implies exists(x: S, y: S) { x != y })
    (exists(x: S, y: S) { x != y } implies S.0 != S.1)
    iff_bool_imp_eq(S.0 != S.1, exists(x: S, y: S) { x != y })
}
