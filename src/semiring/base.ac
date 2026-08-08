from algebra.add_comm_monoid import AddCommMonoid
from algebra.monoid.monoid import Monoid

/// A semiring is like a ring but without additive inverses.
/// It has two operations where addition forms a commutative monoid, multiplication forms a monoid,
/// and multiplication distributes over addition.
typeclass S: Semiring extends AddCommMonoid, Monoid {
    /// Multiplication distributes over addition from the left: `a * (b + c) = (a * b) + (a * c)`.
    distrib_left(a: S, b: S, c: S) {
        a * (b + c) = (a * b) + (a * c)
    }

    /// Multiplication distributes over addition from the right: `(a + b) * c = (a * c) + (b * c)`.
    distrib_right(a: S, b: S, c: S) {
        (a + b) * c = (a * c) + (b * c)
    }

    /// Multiplying by the additive identity yields the additive identity.
    mul_zero_left(a: S) {
        a * S.0 = S.0
    }

    /// Multiplying the additive identity by anything yields the additive identity.
    mul_zero_right(a: S) {
        S.0 * a = S.0
    }
}

theorem trivial_semiring[S: Semiring] {
    S.0 = S.1 implies forall(s: S) { s = S.0 }
}
