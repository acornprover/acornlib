from nat import Nat
from int import Int
from zmod import Zmod, int_mod_rel, zmod_mk, zmod_zero, zmod_one, zmod_add,
    zmod_neg, zmod_mul, zmod_mk_eq_of_rel, zmod_rel_of_mk_eq,
    zmod_mk_carrier, zmod_zero_mk, zmod_one_mk, zmod_add_mk,
    zmod_neg_mk, zmod_mul_mk
from data.int.int_congruence import int_congr_mod_iff_int_mod_rel,
    int_mod_rel_of_congr_mod, int_congr_mod_of_int_mod_rel,
    int_congr_mod_refl, int_congr_mod_add, int_congr_mod_neg,
    int_congr_mod_mul, int_congr_mod_zero_eq, int_congr_mod_zero_iff_divides
from data.int.int_residue import int_has_nat_residue
from data.basic.set import equivalence_class

/// Equality of constructor-normal forms in `Zmod[n]` is exactly integer congruence.
theorem zmod_mk_eq_iff_int_mod_rel(n: Nat, a: Int, b: Int) {
    (zmod_mk(n, a) = zmod_mk(n, b)) = int_mod_rel(n, a, b)
} by {
    if zmod_mk(n, a) = zmod_mk(n, b) {
        zmod_rel_of_mk_eq(n, a, b)
    }
    if int_mod_rel(n, a, b) {
        zmod_mk_eq_of_rel(n, a, b)
    }
}

/// Equality of constructor-normal forms in `Zmod[n]` is exactly method-style integer congruence.
theorem zmod_mk_eq_iff_int_congr_mod(n: Nat, a: Int, b: Int) {
    (zmod_mk(n, a) = zmod_mk(n, b)) = a.congr_mod(b, n)
} by {
    zmod_mk_eq_iff_int_mod_rel(n, a, b)
    int_congr_mod_iff_int_mod_rel(a, b, n)
}

/// Method-style congruence gives equality of residue constructors.
theorem zmod_mk_eq_of_int_congr_mod(n: Nat, a: Int, b: Int) {
    a.congr_mod(b, n) implies zmod_mk(n, a) = zmod_mk(n, b)
} by {
    if a.congr_mod(b, n) {
        int_mod_rel_of_congr_mod(a, b, n)
        zmod_mk_eq_of_rel(n, a, b)
    }
}

/// Equality of residue constructors gives method-style congruence.
theorem int_congr_mod_of_zmod_mk_eq(n: Nat, a: Int, b: Int) {
    zmod_mk(n, a) = zmod_mk(n, b) implies a.congr_mod(b, n)
} by {
    if zmod_mk(n, a) = zmod_mk(n, b) {
        zmod_rel_of_mk_eq(n, a, b)
        int_congr_mod_of_int_mod_rel(a, b, n)
    }
}

/// A constructor is unchanged when its representative is replaced by a congruent integer.
theorem zmod_mk_congr_mod_replace(n: Nat, a: Int, b: Int) {
    a.congr_mod(b, n) implies zmod_mk(n, a) = zmod_mk(n, b)
} by {
    zmod_mk_eq_of_int_congr_mod(n, a, b)
}

/// Equality in `Zmod[0]` constructor form is equality of integer representatives.
theorem zmod_mk_eq_zero_mod_iff_eq(a: Int, b: Int) {
    (zmod_mk(Nat.0, a) = zmod_mk(Nat.0, b)) = (a = b)
} by {
    zmod_mk_eq_iff_int_congr_mod(Nat.0, a, b)
    int_congr_mod_zero_eq(a, b)
}

/// Constructor equality against zero is divisibility by the modulus.
theorem zmod_mk_eq_zero_iff_divides(n: Nat, a: Int) {
    (zmod_mk(n, a) = zmod_zero(n)) = Int.from_nat(n).divides(a)
} by {
    zmod_zero_mk(n)
    zmod_mk_eq_iff_int_congr_mod(n, a, Int.0)
    int_congr_mod_zero_iff_divides(a, n)
}

/// Constructor equality against one is method-style congruence to one.
theorem zmod_mk_eq_one_iff_congr_one(n: Nat, a: Int) {
    (zmod_mk(n, a) = zmod_one(n)) = a.congr_mod(Int.1, n)
} by {
    zmod_one_mk(n)
    zmod_mk_eq_iff_int_congr_mod(n, a, Int.1)
}

/// Addition transports through method-style congruence into `Zmod[n]` equality.
theorem zmod_mk_add_congr_mod(n: Nat, a1: Int, a2: Int, b1: Int, b2: Int) {
    a1.congr_mod(a2, n) and b1.congr_mod(b2, n) implies
    zmod_mk(n, a1 + b1) = zmod_mk(n, a2 + b2)
} by {
    if a1.congr_mod(a2, n) and b1.congr_mod(b2, n) {
        int_congr_mod_add(a1, b1, a2, b2, n)
        (a1 + b1).congr_mod(a2 + b2, n)
        zmod_mk_eq_of_int_congr_mod(n, a1 + b1, a2 + b2)
    }
}

/// Negation transports through method-style congruence into `Zmod[n]` equality.
theorem zmod_mk_neg_congr_mod(n: Nat, a: Int, b: Int) {
    a.congr_mod(b, n) implies zmod_mk(n, -a) = zmod_mk(n, -b)
} by {
    if a.congr_mod(b, n) {
        int_congr_mod_neg(a, b, n)
        (-a).congr_mod(-b, n)
        zmod_mk_eq_of_int_congr_mod(n, -a, -b)
    }
}

/// Multiplication transports through method-style congruence into `Zmod[n]` equality.
theorem zmod_mk_mul_congr_mod(n: Nat, a1: Int, a2: Int, b1: Int, b2: Int) {
    a1.congr_mod(a2, n) and b1.congr_mod(b2, n) implies
    zmod_mk(n, a1 * b1) = zmod_mk(n, a2 * b2)
} by {
    if a1.congr_mod(a2, n) and b1.congr_mod(b2, n) {
        int_congr_mod_mul(a1, b1, a2, b2, n)
        (a1 * b1).congr_mod(a2 * b2, n)
        zmod_mk_eq_of_int_congr_mod(n, a1 * b1, a2 * b2)
    }
}

/// Congruent left addends give equal residue sums.
theorem zmod_add_left_congr_mod(n: Nat, a: Int, b: Int, c: Int) {
    a.congr_mod(b, n) implies
    zmod_add(n, zmod_mk(n, a), zmod_mk(n, c)) =
        zmod_add(n, zmod_mk(n, b), zmod_mk(n, c))
} by {
    if a.congr_mod(b, n) {
        int_congr_mod_refl(c, n)
        zmod_add_mk(n, a, c)
        zmod_add_mk(n, b, c)
        zmod_mk_add_congr_mod(n, a, b, c, c)
    }
}

/// Congruent right addends give equal residue sums.
theorem zmod_add_right_congr_mod(n: Nat, a: Int, b: Int, c: Int) {
    a.congr_mod(b, n) implies
    zmod_add(n, zmod_mk(n, c), zmod_mk(n, a)) =
        zmod_add(n, zmod_mk(n, c), zmod_mk(n, b))
} by {
    if a.congr_mod(b, n) {
        int_congr_mod_refl(c, n)
        zmod_add_mk(n, c, a)
        zmod_add_mk(n, c, b)
        zmod_mk_add_congr_mod(n, c, c, a, b)
    }
}

/// Congruent left factors give equal residue products.
theorem zmod_mul_left_congr_mod(n: Nat, a: Int, b: Int, c: Int) {
    a.congr_mod(b, n) implies
    zmod_mul(n, zmod_mk(n, a), zmod_mk(n, c)) =
        zmod_mul(n, zmod_mk(n, b), zmod_mk(n, c))
} by {
    if a.congr_mod(b, n) {
        int_congr_mod_refl(c, n)
        zmod_mul_mk(n, a, c)
        zmod_mul_mk(n, b, c)
        zmod_mk_mul_congr_mod(n, a, b, c, c)
    }
}

/// Congruent right factors give equal residue products.
theorem zmod_mul_right_congr_mod(n: Nat, a: Int, b: Int, c: Int) {
    a.congr_mod(b, n) implies
    zmod_mul(n, zmod_mk(n, c), zmod_mk(n, a)) =
        zmod_mul(n, zmod_mk(n, c), zmod_mk(n, b))
} by {
    if a.congr_mod(b, n) {
        int_congr_mod_refl(c, n)
        zmod_mul_mk(n, c, a)
        zmod_mul_mk(n, c, b)
        zmod_mk_mul_congr_mod(n, c, c, a, b)
    }
}

/// Negating congruent representatives gives equal residue negations.
theorem zmod_neg_congr_mod(n: Nat, a: Int, b: Int) {
    a.congr_mod(b, n) implies
    zmod_neg(n, zmod_mk(n, a)) = zmod_neg(n, zmod_mk(n, b))
} by {
    if a.congr_mod(b, n) {
        zmod_neg_mk(n, a)
        zmod_neg_mk(n, b)
        zmod_mk_neg_congr_mod(n, a, b)
    }
}

/// The carrier of a constructor is its integer congruence class.
theorem zmod_mk_carrier_normal_form(n: Nat, a: Int) {
    zmod_mk(n, a).carrier = equivalence_class(int_mod_rel(n), a)
} by {
    zmod_mk_carrier(n, a)
}

/// Every residue class modulo a positive modulus has a natural-number representative.
theorem zmod_has_nat_representative(n: Nat, x: Zmod[n]) {
    n != Nat.0 implies exists(r: Nat) { zmod_mk(n, Int.from_nat(r)) = x }
} by {
    if n != Nat.0 {
        let a: Int satisfy { zmod_mk(n, a) = x }
        int_has_nat_residue(a, n)
        let r: Nat satisfy { int_mod_rel(n, a, Int.from_nat(r)) }
        zmod_mk_eq_of_rel(n, a, Int.from_nat(r))
        zmod_mk(n, a) = zmod_mk(n, Int.from_nat(r))
        zmod_mk(n, Int.from_nat(r)) = x
        exists(r0: Nat) { zmod_mk(n, Int.from_nat(r0)) = x }
    }
}
