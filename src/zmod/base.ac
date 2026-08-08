from nat import Nat
from int import Int, add_assoc, add_comm, add_neg, add_zero_left, add_zero_right,
    mul_assoc, mul_comm, mul_distrib_left, mul_distrib_right, mul_one_left,
    mul_one_right, mul_zero_left, mul_zero_right, ten_sub_one_eq_nine,
    three_mul_three_eq_nine, sub_zero_right
from algebra.add import Add
from algebra.zero import Zero
from algebra.neg import Neg
from algebra.mul import Mul
from algebra.one import One
from data.basic.relation_basic import is_reflexive, is_symmetric, is_transitive, is_equivalence
from data.basic.relation_transport import is_unary_congruence, is_binary_congruence
from data.basic.set import Set, equivalence_class, equivalence_class_eq_of_equiv,
    equivalence_of_equivalence_class_eq
from data.basic.equivalence import is_equivalence_class, equivalence_class_is_equivalence_class,
    QuotientRelation, Quotient, QuotientOver, quotient_mk, quotient_over_mk,
    quotient_over_mk_eq_iff_rel, quotient_over_binary_op, quotient_over_binary_op_projection,
    quotient_over_binary_op_projection_left, quotient_over_binary_op_projection_right,
    quotient_over_binary_op_projection_identity_left,
    quotient_over_binary_op_projection_identity_right, quotient_over_const,
    quotient_over_const_projection, quotient_over_unary_op, quotient_over_unary_op_projection,
    quotient_over_unary_op_projection_compatible, quotient_relation_new_round_trip,
    quotient_mk_eq_iff_rel, quotient_projection, quotient_projection_eq_mk,
    quotient_projection_eq_of_rel, quotient_unary_respects, quotient_unary_projection_compatible,
    quotient_unary_respects_is_unary_congruence, quotient_binary_respects,
    quotient_binary_projection_compatible, quotient_binary_projection_compatible_left,
    quotient_binary_projection_compatible_right, quotient_binary_respects_is_binary_congruence

/// True if two integers are congruent modulo `n`.
define int_mod_rel(n: Nat, a: Int, b: Int) -> Bool {
    Int.from_nat(n).divides(a - b)
}

/// The addition operation on integers, as a named binary operation.
define int_add_op(a: Int, b: Int) -> Int {
    a + b
}

/// The multiplication operation on integers, as a named binary operation.
define int_mul_op(a: Int, b: Int) -> Int {
    a * b
}

/// The negation operation on integers, as a named unary operation.
define int_neg_op(a: Int) -> Int {
    -a
}

/// The zero representative for integer residue classes.
let int_zero_rep: Int = Int.0

/// The one representative for integer residue classes.
let int_one_rep: Int = Int.1

/// Congruence modulo zero is equality of integers.
theorem int_mod_rel_zero_eq(a: Int, b: Int) {
    int_mod_rel(Nat.0, a, b) = (a = b)
} by {
    if int_mod_rel(Nat.0, a, b) {
        let d: Int satisfy { d * Int.from_nat(Nat.0) = a - b }
        d * Int.from_nat(Nat.0) = Int.0
        a - b = Int.0
        a = b
    }
    if a = b {
        a - b = Int.0
        Int.0 * Int.from_nat(Nat.0) = a - b
        Int.from_nat(Nat.0).divides(a - b)
        int_mod_rel(Nat.0, a, b)
    }
    int_mod_rel(Nat.0, a, b) = (a = b)
}

/// Congruence modulo `n` is an equivalence relation.
theorem int_mod_rel_is_equivalence(n: Nat) {
    is_equivalence(int_mod_rel(n))
} by {
    forall(a: Int) {
        a - a = Int.0
        Int.0 * Int.from_nat(n) = a - a
        Int.from_nat(n).divides(a - a)
        int_mod_rel(n, a, a)
    }
    is_reflexive(int_mod_rel(n))
    forall(a: Int, b: Int) {
        if int_mod_rel(n, a, b) {
            let d: Int satisfy { d * Int.from_nat(n) = a - b }
            -d * Int.from_nat(n) = -(d * Int.from_nat(n))
            -(a - b) = b - a
            -d * Int.from_nat(n) = b - a
            Int.from_nat(n).divides(b - a)
            int_mod_rel(n, b, a)
        }
    }
    is_symmetric(int_mod_rel(n))
    forall(a: Int, b: Int, c: Int) {
        if int_mod_rel(n, a, b) and int_mod_rel(n, b, c) {
            let d: Int satisfy { d * Int.from_nat(n) = a - b }
            let e: Int satisfy { e * Int.from_nat(n) = b - c }
            (d + e) * Int.from_nat(n) = d * Int.from_nat(n) + e * Int.from_nat(n)
            d * Int.from_nat(n) + e * Int.from_nat(n) = (a - b) + (b - c)
            (a - b) + (b - c) = (a + -b) + (b + -c)
            (a + -b) + (b + -c) = a + (-b + b) + -c
            -b + b = Int.0
            a + (-b + b) + -c = a + Int.0 + -c
            a + Int.0 = a
            a + Int.0 + -c = a + -c
            a + -c = a - c
            (a - b) + (b - c) = a - c
            (d + e) * Int.from_nat(n) = a - c
            Int.from_nat(n).divides(a - c)
            int_mod_rel(n, a, c)
        }
    }
    is_transitive(int_mod_rel(n))
    is_equivalence(int_mod_rel(n))
}

/// The quotient relation associated to congruence modulo `n`.
let int_mod_quotient_relation(n: Nat) -> result: QuotientRelation[Int] satisfy {
    QuotientRelation[Int].new(int_mod_rel(n)) = Option.some(result)
} by {
    int_mod_rel_is_equivalence(n)
}

/// The quotient relation associated to congruence modulo `n` has the expected relation.
theorem int_mod_quotient_relation_rel(n: Nat) {
    int_mod_quotient_relation(n).rel = int_mod_rel(n)
} by {
    QuotientRelation[Int].new(int_mod_rel(n)) = Option.some(int_mod_quotient_relation(n))
    quotient_relation_new_round_trip(int_mod_rel(n), int_mod_quotient_relation(n))
}

/// The generic quotient class of an integer modulo `n`.
define int_mod_quotient_mk(n: Nat, a: Int) -> Quotient[Int] {
    quotient_mk(int_mod_quotient_relation(n), a)
}

/// Equality of generic quotient representatives is congruence modulo `n`.
theorem int_mod_quotient_mk_eq_iff_rel(n: Nat, a: Int, b: Int) {
    (int_mod_quotient_mk(n, a) = int_mod_quotient_mk(n, b)) = int_mod_rel(n, a, b)
} by {
    int_mod_quotient_relation_rel(n)
    quotient_mk_eq_iff_rel(int_mod_quotient_relation(n), a, b)
}

/// The generic quotient projection for congruence modulo `n`.
define int_mod_projection(n: Nat, a: Int) -> Quotient[Int] {
    quotient_projection(int_mod_quotient_relation(n), a)
}

/// The relation-indexed quotient class of an integer modulo `n`.
define int_mod_over_mk(n: Nat, a: Int) -> QuotientOver[Int] {
    quotient_over_mk(int_mod_quotient_relation(n), a)
}

/// The generic quotient projection is the generic quotient constructor.
theorem int_mod_projection_eq_mk(n: Nat, a: Int) {
    int_mod_projection(n, a) = int_mod_quotient_mk(n, a)
} by {
    quotient_projection_eq_mk(int_mod_quotient_relation(n), a)
}

/// Equality of relation-indexed quotient representatives is congruence modulo `n`.
theorem int_mod_over_mk_eq_iff_rel(n: Nat, a: Int, b: Int) {
    (int_mod_over_mk(n, a) = int_mod_over_mk(n, b)) = int_mod_rel(n, a, b)
} by {
    int_mod_quotient_relation_rel(n)
    quotient_over_mk_eq_iff_rel(int_mod_quotient_relation(n), a, b)
}

/// Related integers have the same projection modulo `n`.
theorem int_mod_projection_eq_of_rel(n: Nat, a: Int, b: Int) {
    int_mod_rel(n, a, b) implies int_mod_projection(n, a) = int_mod_projection(n, b)
} by {
    if int_mod_rel(n, a, b) {
        int_mod_quotient_relation_rel(n)
        quotient_projection_eq_of_rel(int_mod_quotient_relation(n), a, b)
    }
}

/// Congruence modulo `n` is compatible with addition.
theorem int_mod_rel_add_compatible(n: Nat, a1: Int, a2: Int, b1: Int, b2: Int) {
    int_mod_rel(n, a1, a2) and int_mod_rel(n, b1, b2) implies
    int_mod_rel(n, a1 + b1, a2 + b2)
} by {
    if int_mod_rel(n, a1, a2) and int_mod_rel(n, b1, b2) {
        let m = Int.from_nat(n)
        let d: Int satisfy { d * m = a1 - a2 }
        let e: Int satisfy { e * m = b1 - b2 }
        (d + e) * m = d * m + e * m
        d * m + e * m = (a1 - a2) + (b1 - b2)
        (a1 - a2) + (b1 - b2) = (a1 + -a2) + (b1 + -b2)
        (a1 + -a2) + (b1 + -b2) = a1 + b1 + (-a2 + -b2)
        -a2 + -b2 = -(a2 + b2)
        a1 + b1 + (-a2 + -b2) = a1 + b1 + -(a2 + b2)
        a1 + b1 + -(a2 + b2) = (a1 + b1) - (a2 + b2)
        (a1 - a2) + (b1 - b2) = (a1 + b1) - (a2 + b2)
        (d + e) * m = (a1 + b1) - (a2 + b2)
        m.divides((a1 + b1) - (a2 + b2))
        int_mod_rel(n, a1 + b1, a2 + b2)
    }
}

/// Congruence modulo `n` respects integer addition as a binary operation.
theorem int_mod_add_respects(n: Nat) {
    quotient_binary_respects(int_mod_quotient_relation(n), int_add_op)
} by {
    forall(a1: Int, a2: Int, b1: Int, b2: Int) {
        if int_mod_quotient_relation(n).rel(a1, a2) and int_mod_quotient_relation(n).rel(b1, b2) {
            int_mod_quotient_relation_rel(n)
            int_mod_rel(n, a1, a2)
            int_mod_rel(n, b1, b2)
            int_mod_rel_add_compatible(n, a1, a2, b1, b2)
            int_mod_rel(n, a1 + b1, a2 + b2)
            int_add_op(a1, b1) = a1 + b1
            int_add_op(a2, b2) = a2 + b2
            int_mod_quotient_relation(n).rel(int_add_op(a1, b1), int_add_op(a2, b2))
        }
    }
}

/// Congruence modulo `n` is a binary congruence for integer addition.
theorem int_mod_add_is_binary_congruence(n: Nat) {
    is_binary_congruence(int_mod_quotient_relation(n).rel, int_add_op)
} by {
    int_mod_add_respects(n)
    quotient_binary_respects_is_binary_congruence(int_mod_quotient_relation(n), int_add_op)
}

/// Addition of related representatives has the same projection modulo `n`.
theorem int_mod_projection_add_compatible(n: Nat, a1: Int, a2: Int, b1: Int, b2: Int) {
    int_mod_rel(n, a1, a2) and int_mod_rel(n, b1, b2) implies
    int_mod_projection(n, a1 + b1) = int_mod_projection(n, a2 + b2)
} by {
    if int_mod_rel(n, a1, a2) and int_mod_rel(n, b1, b2) {
        int_mod_quotient_relation_rel(n)
        int_mod_add_respects(n)
        quotient_binary_projection_compatible(int_mod_quotient_relation(n), int_add_op, a1, a2, b1, b2)
        int_add_op(a1, b1) = a1 + b1
        int_add_op(a2, b2) = a2 + b2
    }
}

/// Addition of related left representatives has the same projection modulo `n`.
theorem int_mod_projection_add_compatible_left(n: Nat, a1: Int, a2: Int, b: Int) {
    int_mod_rel(n, a1, a2) implies
    int_mod_projection(n, a1 + b) = int_mod_projection(n, a2 + b)
} by {
    if int_mod_rel(n, a1, a2) {
        int_mod_quotient_relation_rel(n)
        int_mod_add_respects(n)
        quotient_binary_projection_compatible_left(int_mod_quotient_relation(n), int_add_op, a1, a2, b)
        int_add_op(a1, b) = a1 + b
        int_add_op(a2, b) = a2 + b
    }
}

/// Addition of related right representatives has the same projection modulo `n`.
theorem int_mod_projection_add_compatible_right(n: Nat, a: Int, b1: Int, b2: Int) {
    int_mod_rel(n, b1, b2) implies
    int_mod_projection(n, a + b1) = int_mod_projection(n, a + b2)
} by {
    if int_mod_rel(n, b1, b2) {
        int_mod_quotient_relation_rel(n)
        int_mod_add_respects(n)
        quotient_binary_projection_compatible_right(int_mod_quotient_relation(n), int_add_op, a, b1, b2)
        int_add_op(a, b1) = a + b1
        int_add_op(a, b2) = a + b2
    }
}

/// Addition of integer residue classes in the relation-indexed quotient.
define int_mod_over_add(n: Nat) -> (QuotientOver[Int], QuotientOver[Int]) -> QuotientOver[Int] {
    quotient_over_binary_op(int_add_op)
}

/// Addition in the relation-indexed quotient agrees with addition of representatives.
theorem int_mod_over_add_projection(n: Nat, a: Int, b: Int) {
    int_mod_over_add(n, int_mod_over_mk(n, a), int_mod_over_mk(n, b)) =
        int_mod_over_mk(n, a + b)
} by {
    int_mod_add_respects(n)
    quotient_over_binary_op_projection(int_mod_quotient_relation(n), int_add_op, a, b)
    int_add_op(a, b) = a + b
}

/// Addition in the relation-indexed quotient respects related left representatives.
theorem int_mod_over_add_projection_left(n: Nat, a1: Int, a2: Int, b: Int) {
    int_mod_rel(n, a1, a2) implies
    int_mod_over_add(n, int_mod_over_mk(n, a1), int_mod_over_mk(n, b)) =
        int_mod_over_add(n, int_mod_over_mk(n, a2), int_mod_over_mk(n, b))
} by {
    if int_mod_rel(n, a1, a2) {
        int_mod_quotient_relation_rel(n)
        int_mod_add_respects(n)
        quotient_over_binary_op_projection_left(int_mod_quotient_relation(n), int_add_op, a1, a2, b)
    }
}

/// Addition in the relation-indexed quotient respects related right representatives.
theorem int_mod_over_add_projection_right(n: Nat, a: Int, b1: Int, b2: Int) {
    int_mod_rel(n, b1, b2) implies
    int_mod_over_add(n, int_mod_over_mk(n, a), int_mod_over_mk(n, b1)) =
        int_mod_over_add(n, int_mod_over_mk(n, a), int_mod_over_mk(n, b2))
} by {
    if int_mod_rel(n, b1, b2) {
        int_mod_quotient_relation_rel(n)
        int_mod_add_respects(n)
        quotient_over_binary_op_projection_right(int_mod_quotient_relation(n), int_add_op, a, b1, b2)
    }
}

/// Congruence modulo `n` is compatible with negation.
theorem int_mod_rel_neg_compatible(n: Nat, a: Int, b: Int) {
    int_mod_rel(n, a, b) implies int_mod_rel(n, -a, -b)
} by {
    if int_mod_rel(n, a, b) {
        let m = Int.from_nat(n)
        let d: Int satisfy { d * m = a - b }
        -d * m = -(d * m)
        -(a - b) = b - a
        -a - -b = -a + --b
        --b = b
        -a + --b = -a + b
        -a + b = b + -a
        b + -a = b - a
        -a - -b = b - a
        -d * m = b - a
        -d * m = -a - -b
        m.divides(-a - -b)
        int_mod_rel(n, -a, -b)
    }
}

/// Congruence modulo `n` respects integer negation as a unary operation.
theorem int_mod_neg_respects(n: Nat) {
    quotient_unary_respects(int_mod_quotient_relation(n), int_neg_op)
} by {
    forall(a: Int, b: Int) {
        if int_mod_quotient_relation(n).rel(a, b) {
            int_mod_quotient_relation_rel(n)
            int_mod_rel(n, a, b)
            int_mod_rel_neg_compatible(n, a, b)
            int_mod_rel(n, -a, -b)
            int_neg_op(a) = -a
            int_neg_op(b) = -b
            int_mod_quotient_relation(n).rel(int_neg_op(a), int_neg_op(b))
        }
    }
}

/// Congruence modulo `n` is a unary congruence for integer negation.
theorem int_mod_neg_is_unary_congruence(n: Nat) {
    is_unary_congruence(int_mod_quotient_relation(n).rel, int_neg_op)
} by {
    int_mod_neg_respects(n)
    quotient_unary_respects_is_unary_congruence(int_mod_quotient_relation(n), int_neg_op)
}

/// Negation of related representatives has the same projection modulo `n`.
theorem int_mod_projection_neg_compatible(n: Nat, a: Int, b: Int) {
    int_mod_rel(n, a, b) implies int_mod_projection(n, -a) = int_mod_projection(n, -b)
} by {
    if int_mod_rel(n, a, b) {
        int_mod_quotient_relation_rel(n)
        int_mod_neg_respects(n)
        quotient_unary_projection_compatible(int_mod_quotient_relation(n), int_neg_op, a, b)
        int_neg_op(a) = -a
        int_neg_op(b) = -b
    }
}

/// Negation of integer residue classes in the relation-indexed quotient.
define int_mod_over_neg(n: Nat) -> QuotientOver[Int] -> QuotientOver[Int] {
    quotient_over_unary_op(int_mod_quotient_relation(n), int_neg_op)
}

/// Negation in the relation-indexed quotient agrees with negation of representatives.
theorem int_mod_over_neg_projection(n: Nat, a: Int) {
    int_mod_over_neg(n, int_mod_over_mk(n, a)) = int_mod_over_mk(n, -a)
} by {
    int_mod_neg_respects(n)
    quotient_over_unary_op_projection(int_mod_quotient_relation(n), int_neg_op, a)
    int_neg_op(a) = -a
}

/// Negation in the relation-indexed quotient respects related representatives.
theorem int_mod_over_neg_projection_compatible(n: Nat, a: Int, b: Int) {
    int_mod_rel(n, a, b) implies
    int_mod_over_neg(n, int_mod_over_mk(n, a)) = int_mod_over_neg(n, int_mod_over_mk(n, b))
} by {
    if int_mod_rel(n, a, b) {
        int_mod_quotient_relation_rel(n)
        int_mod_neg_respects(n)
        quotient_over_unary_op_projection_compatible(int_mod_quotient_relation(n), int_neg_op, a, b)
    }
}

/// Congruence modulo `n` is compatible with multiplication.
theorem int_mod_rel_mul_compatible(n: Nat, a1: Int, a2: Int, b1: Int, b2: Int) {
    int_mod_rel(n, a1, a2) and int_mod_rel(n, b1, b2) implies
    int_mod_rel(n, a1 * b1, a2 * b2)
} by {
    if int_mod_rel(n, a1, a2) and int_mod_rel(n, b1, b2) {
        let m = Int.from_nat(n)
        let d: Int satisfy { d * m = a1 - a2 }
        let e: Int satisfy { e * m = b1 - b2 }
        (d * b1 + a2 * e) * m = d * b1 * m + a2 * e * m
        d * b1 * m = d * (b1 * m)
        b1 * m = m * b1
        d * (b1 * m) = d * (m * b1)
        d * (m * b1) = d * m * b1
        d * b1 * m = d * m * b1
        d * m * b1 = (a1 - a2) * b1
        a2 * e * m = a2 * (e * m)
        a2 * (e * m) = a2 * (b1 - b2)
        (d * b1 + a2 * e) * m = (a1 - a2) * b1 + a2 * (b1 - b2)
        (a1 - a2) * b1 = a1 * b1 - a2 * b1
        a2 * (b1 - b2) = a2 * b1 - a2 * b2
        let x = a1 * b1
        let y = a2 * b1
        let z = a2 * b2
        x - y + (y - z) = (x + -y) + (y + -z)
        (x + -y) + (y + -z) = x + (-y + y) + -z
        -y + y = Int.0
        x + (-y + y) + -z = x + Int.0 + -z
        x + Int.0 = x
        x + Int.0 + -z = x + -z
        x + -z = x - z
        x - y + (y - z) = x - z
        (a1 * b1 - a2 * b1) + (a2 * b1 - a2 * b2) = a1 * b1 - a2 * b2
        (a1 - a2) * b1 + a2 * (b1 - b2) = a1 * b1 - a2 * b2
        (d * b1 + a2 * e) * m = a1 * b1 - a2 * b2
        m.divides(a1 * b1 - a2 * b2)
        int_mod_rel(n, a1 * b1, a2 * b2)
    }
}

/// Multiplying on the right preserves the congruence from 10 to 1 modulo 3.
theorem int_mod_rel_mul_right_self_mod3(x: Int) {
    int_mod_rel(Nat.3, Int.10, Int.1) implies int_mod_rel(Nat.3, Int.10 * x, Int.1 * x)
} by {
    int_mod_rel_mul_compatible(Nat.3, Int.10, Int.1, x, x)
}

/// One decimal-read step is congruent to adding the read digit modulo 3,
/// assuming 10 is congruent to 1 modulo 3.
theorem int_read_mod3_step_if_ten_mod_one(x: Int, d: Int) {
    int_mod_rel(Nat.3, Int.10, Int.1) implies int_mod_rel(Nat.3, x.read(d), x + d)
} by {
    x.read(d) = Int.10 * x + d
    int_mod_rel(Nat.3, Int.10 * x, Int.1 * x)
    int_mod_rel(Nat.3, d, d)
    int_mod_rel_add_compatible(Nat.3, Int.10 * x, Int.1 * x, d, d)
    Int.1 * x = x
    int_mod_rel(Nat.3, x.read(d), x + d)
}

/// Ten is congruent to one modulo three.
theorem int_mod_rel_ten_one_mod3 {
    int_mod_rel(Nat.3, Int.10, Int.1)
} by {
    ten_sub_one_eq_nine
    three_mul_three_eq_nine
    Int.3.divides(Int.3)
    (Int.3 * Int.1).divides(Int.3 * Int.3)
    Int.3 * Int.1 = Int.3
    Int.3.divides(Int.9)
    Int.from_nat(Nat.3) = Int.3
    Int.from_nat(Nat.3).divides(Int.10 - Int.1)
}

/// One decimal-read step is congruent to adding the read digit modulo 3.
theorem int_read_mod3_step(x: Int, d: Int) {
    int_mod_rel(Nat.3, x.read(d), x + d)
} by {
    int_mod_rel_ten_one_mod3
    int_read_mod3_step_if_ten_mod_one(x, d)
}

/// Reading the same decimal digit preserves congruence modulo 3.
theorem int_read_mod3_compatible(x: Int, y: Int, d: Int) {
    int_mod_rel(Nat.3, x, y) implies int_mod_rel(Nat.3, x.read(d), y.read(d))
} by {
    int_read_mod3_step(x, d)
    int_read_mod3_step(y, d)
    int_mod_rel(Nat.3, d, d)
    int_mod_rel_add_compatible(Nat.3, x, y, d, d)
    int_mod_rel(Nat.3, x + d, y + d)
    int_mod_rel_is_equivalence(Nat.3)
    int_mod_rel(Nat.3, x.read(d), x + d)
    int_mod_rel(Nat.3, y.read(d), y + d)
    int_mod_rel(Nat.3, x + d, y.read(d))
    int_mod_rel(Nat.3, x.read(d), y.read(d))
}

/// Congruence to zero modulo `n` is divisibility by `n`.
theorem int_mod_rel_zero_iff_divides(n: Nat, a: Int) {
    int_mod_rel(n, a, Int.0) = Int.from_nat(n).divides(a)
} by {
    sub_zero_right(a)
    a - Int.0 = a
    int_mod_rel(n, a, Int.0) = Int.from_nat(n).divides(a - Int.0)
    int_mod_rel(n, a, Int.0) = Int.from_nat(n).divides(a)
}

/// Congruence to zero modulo 3 is divisibility by the integer 3.
theorem int_mod_rel_zero_iff_int_three_divides(a: Int) {
    int_mod_rel(Nat.3, a, Int.0) = Int.3.divides(a)
} by {
    int_mod_rel_zero_iff_divides(Nat.3, a)
    Int.from_nat(Nat.3) = Int.3
    int_mod_rel(Nat.3, a, Int.0) = Int.3.divides(a)
}

/// Congruence modulo `n` respects integer multiplication as a binary operation.
theorem int_mod_mul_respects(n: Nat) {
    quotient_binary_respects(int_mod_quotient_relation(n), int_mul_op)
} by {
    forall(a1: Int, a2: Int, b1: Int, b2: Int) {
        if int_mod_quotient_relation(n).rel(a1, a2) and int_mod_quotient_relation(n).rel(b1, b2) {
            int_mod_quotient_relation_rel(n)
            int_mod_rel(n, a1, a2)
            int_mod_rel(n, b1, b2)
            int_mod_rel_mul_compatible(n, a1, a2, b1, b2)
            int_mod_rel(n, a1 * b1, a2 * b2)
            int_mul_op(a1, b1) = a1 * b1
            int_mul_op(a2, b2) = a2 * b2
            int_mod_quotient_relation(n).rel(int_mul_op(a1, b1), int_mul_op(a2, b2))
        }
    }
}

/// Congruence modulo `n` is a binary congruence for integer multiplication.
theorem int_mod_mul_is_binary_congruence(n: Nat) {
    is_binary_congruence(int_mod_quotient_relation(n).rel, int_mul_op)
} by {
    int_mod_mul_respects(n)
    quotient_binary_respects_is_binary_congruence(int_mod_quotient_relation(n), int_mul_op)
}

/// Multiplication of related representatives has the same projection modulo `n`.
theorem int_mod_projection_mul_compatible(n: Nat, a1: Int, a2: Int, b1: Int, b2: Int) {
    int_mod_rel(n, a1, a2) and int_mod_rel(n, b1, b2) implies
    int_mod_projection(n, a1 * b1) = int_mod_projection(n, a2 * b2)
} by {
    if int_mod_rel(n, a1, a2) and int_mod_rel(n, b1, b2) {
        int_mod_quotient_relation_rel(n)
        int_mod_mul_respects(n)
        quotient_binary_projection_compatible(int_mod_quotient_relation(n), int_mul_op, a1, a2, b1, b2)
        int_mul_op(a1, b1) = a1 * b1
        int_mul_op(a2, b2) = a2 * b2
    }
}

/// Multiplication of related left representatives has the same projection modulo `n`.
theorem int_mod_projection_mul_compatible_left(n: Nat, a1: Int, a2: Int, b: Int) {
    int_mod_rel(n, a1, a2) implies
    int_mod_projection(n, a1 * b) = int_mod_projection(n, a2 * b)
} by {
    if int_mod_rel(n, a1, a2) {
        int_mod_quotient_relation_rel(n)
        int_mod_mul_respects(n)
        quotient_binary_projection_compatible_left(int_mod_quotient_relation(n), int_mul_op, a1, a2, b)
        int_mul_op(a1, b) = a1 * b
        int_mul_op(a2, b) = a2 * b
    }
}

/// Multiplication of related right representatives has the same projection modulo `n`.
theorem int_mod_projection_mul_compatible_right(n: Nat, a: Int, b1: Int, b2: Int) {
    int_mod_rel(n, b1, b2) implies
    int_mod_projection(n, a * b1) = int_mod_projection(n, a * b2)
} by {
    if int_mod_rel(n, b1, b2) {
        int_mod_quotient_relation_rel(n)
        int_mod_mul_respects(n)
        quotient_binary_projection_compatible_right(int_mod_quotient_relation(n), int_mul_op, a, b1, b2)
        int_mul_op(a, b1) = a * b1
        int_mul_op(a, b2) = a * b2
    }
}

/// Multiplication of integer residue classes in the relation-indexed quotient.
define int_mod_over_mul(n: Nat) -> (QuotientOver[Int], QuotientOver[Int]) -> QuotientOver[Int] {
    quotient_over_binary_op(int_mul_op)
}

/// Multiplication in the relation-indexed quotient agrees with multiplication of representatives.
theorem int_mod_over_mul_projection(n: Nat, a: Int, b: Int) {
    int_mod_over_mul(n, int_mod_over_mk(n, a), int_mod_over_mk(n, b)) =
        int_mod_over_mk(n, a * b)
} by {
    int_mod_mul_respects(n)
    quotient_over_binary_op_projection(int_mod_quotient_relation(n), int_mul_op, a, b)
    int_mul_op(a, b) = a * b
}

/// Multiplication in the relation-indexed quotient respects related left representatives.
theorem int_mod_over_mul_projection_left(n: Nat, a1: Int, a2: Int, b: Int) {
    int_mod_rel(n, a1, a2) implies
    int_mod_over_mul(n, int_mod_over_mk(n, a1), int_mod_over_mk(n, b)) =
        int_mod_over_mul(n, int_mod_over_mk(n, a2), int_mod_over_mk(n, b))
} by {
    if int_mod_rel(n, a1, a2) {
        int_mod_quotient_relation_rel(n)
        int_mod_mul_respects(n)
        quotient_over_binary_op_projection_left(int_mod_quotient_relation(n), int_mul_op, a1, a2, b)
    }
}

/// Multiplication in the relation-indexed quotient respects related right representatives.
theorem int_mod_over_mul_projection_right(n: Nat, a: Int, b1: Int, b2: Int) {
    int_mod_rel(n, b1, b2) implies
    int_mod_over_mul(n, int_mod_over_mk(n, a), int_mod_over_mk(n, b1)) =
        int_mod_over_mul(n, int_mod_over_mk(n, a), int_mod_over_mk(n, b2))
} by {
    if int_mod_rel(n, b1, b2) {
        int_mod_quotient_relation_rel(n)
        int_mod_mul_respects(n)
        quotient_over_binary_op_projection_right(int_mod_quotient_relation(n), int_mul_op, a, b1, b2)
    }
}

/// The zero residue class in the relation-indexed quotient.
define int_mod_over_zero(n: Nat) -> QuotientOver[Int] {
    quotient_over_const(int_mod_quotient_relation(n), int_zero_rep)
}

/// The one residue class in the relation-indexed quotient.
define int_mod_over_one(n: Nat) -> QuotientOver[Int] {
    quotient_over_const(int_mod_quotient_relation(n), int_one_rep)
}

/// The zero residue class is represented by zero.
theorem int_mod_over_zero_projection(n: Nat) {
    int_mod_over_zero(n) = int_mod_over_mk(n, Int.0)
} by {
    int_zero_rep = Int.0
    quotient_over_const_projection(int_mod_quotient_relation(n), int_zero_rep)
}

/// The one residue class is represented by one.
theorem int_mod_over_one_projection(n: Nat) {
    int_mod_over_one(n) = int_mod_over_mk(n, Int.1)
} by {
    int_one_rep = Int.1
    quotient_over_const_projection(int_mod_quotient_relation(n), int_one_rep)
}

/// Zero is a left identity for addition of projected integer residue classes.
theorem int_mod_over_add_zero_left(n: Nat, a: Int) {
    int_mod_over_add(n, int_mod_over_zero(n), int_mod_over_mk(n, a)) = int_mod_over_mk(n, a)
} by {
    int_mod_add_respects(n)
    int_zero_rep = Int.0
    int_add_op(int_zero_rep, a) = a
    quotient_over_binary_op_projection_identity_left(int_mod_quotient_relation(n), int_add_op, int_zero_rep, a)
}

/// Zero is a right identity for addition of projected integer residue classes.
theorem int_mod_over_add_zero_right(n: Nat, a: Int) {
    int_mod_over_add(n, int_mod_over_mk(n, a), int_mod_over_zero(n)) = int_mod_over_mk(n, a)
} by {
    int_mod_add_respects(n)
    int_zero_rep = Int.0
    int_add_op(a, int_zero_rep) = a
    quotient_over_binary_op_projection_identity_right(int_mod_quotient_relation(n), int_add_op, int_zero_rep, a)
}

/// One is a left identity for multiplication of projected integer residue classes.
theorem int_mod_over_mul_one_left(n: Nat, a: Int) {
    int_mod_over_mul(n, int_mod_over_one(n), int_mod_over_mk(n, a)) = int_mod_over_mk(n, a)
} by {
    int_mod_mul_respects(n)
    int_one_rep = Int.1
    int_mul_op(int_one_rep, a) = a
    quotient_over_binary_op_projection_identity_left(int_mod_quotient_relation(n), int_mul_op, int_one_rep, a)
}

/// One is a right identity for multiplication of projected integer residue classes.
theorem int_mod_over_mul_one_right(n: Nat, a: Int) {
    int_mod_over_mul(n, int_mod_over_mk(n, a), int_mod_over_one(n)) = int_mod_over_mk(n, a)
} by {
    int_mod_mul_respects(n)
    int_one_rep = Int.1
    int_mul_op(a, int_one_rep) = a
    quotient_over_binary_op_projection_identity_right(int_mod_quotient_relation(n), int_mul_op, int_one_rep, a)
}

/// Negation is a left additive inverse for projected integer residue classes.
theorem int_mod_over_add_neg_left(n: Nat, a: Int) {
    int_mod_over_add(n, int_mod_over_neg(n, int_mod_over_mk(n, a)), int_mod_over_mk(n, a)) =
        int_mod_over_zero(n)
} by {
    int_mod_neg_respects(n)
    int_mod_add_respects(n)
    int_mod_over_neg_projection(n, a)
    int_mod_over_add_projection(n, -a, a)
    int_add_op(-a, a) = Int.0
    int_mod_over_zero_projection(n)
}

/// Negation is a right additive inverse for projected integer residue classes.
theorem int_mod_over_add_neg_right(n: Nat, a: Int) {
    int_mod_over_add(n, int_mod_over_mk(n, a), int_mod_over_neg(n, int_mod_over_mk(n, a))) =
        int_mod_over_zero(n)
} by {
    int_mod_neg_respects(n)
    int_mod_add_respects(n)
    int_mod_over_neg_projection(n, a)
    int_mod_over_add_projection(n, a, -a)
    int_add_op(a, -a) = Int.0
    int_mod_over_zero_projection(n)
}

/// Integer residue classes modulo `n`.
///
/// When `n = 0`, this is the quotient of the integers by equality, so it is
/// equivalent to the integers.
structure Zmod[n: Nat] {
    carrier: Set[Int]
} constraint {
    is_equivalence_class(int_mod_rel(n), carrier)
}

/// The residue class of an integer modulo `n`.
let zmod_mk(n: Nat, a: Int) -> result: Zmod[n] satisfy {
    Zmod[n].new(equivalence_class(int_mod_rel(n), a)) = Option.some(result)
} by {
    let c = equivalence_class(int_mod_rel(n), a)
    equivalence_class_is_equivalence_class(int_mod_rel(n), a)
    is_equivalence_class(int_mod_rel(n), c)
}

/// Identity witness used to exercise top-level dependent `let ... satisfy`.
let zmod_self(n: Nat, x: Zmod[n]) -> result: Zmod[n] satisfy {
    result = x
} by {
    x = x
}

attributes Zmod[n: Nat] {
    /// The modulus of this residue class.
    define modulus(self) -> Nat {
        n
    }

    /// Rewrap through a top-level dependent satisfy function.
    define again(self) -> Zmod[n] {
        zmod_self(n, self)
    }
}

theorem new_round_trip(n: Nat, c: Set[Int], z: Zmod[n]) {
    Zmod[n].new(c) = Option.some(z) implies z.carrier = c
}

theorem new_self(n: Nat, z: Zmod[n]) {
    Zmod[n].new(z.carrier) = Option.some(z)
}

theorem ext(n: Nat, a: Zmod[n], b: Zmod[n]) {
    a.carrier = b.carrier implies a = b
} by {
    if a.carrier = b.carrier {
        Zmod[n].new(a.carrier) = Option.some(a)
        Zmod[n].new(b.carrier) = Option.some(b)
        Option.some(a) = Option.some(b)
        some_injective[Zmod[n]](a, b)
    }
}

theorem mk_new_self(n: Nat, a: Int) {
    Zmod[n].new(equivalence_class(int_mod_rel(n), a)) = Option.some(zmod_mk(n, a))
}

/// Every residue class has an integer representative.
theorem zmod_mk_surjective(n: Nat, z: Zmod[n]) {
    exists(a: Int) {
        zmod_mk(n, a) = z
    }
} by {
    is_equivalence_class(int_mod_rel(n), z.carrier)
    let a: Int satisfy {
        z.carrier = equivalence_class(int_mod_rel(n), a)
    }
    let za = zmod_mk(n, a)
    Zmod[n].new(z.carrier) = Option.some(z)
    Zmod[n].new(equivalence_class(int_mod_rel(n), a)) = Option.some(za)
    Zmod[n].new(z.carrier) = Zmod[n].new(equivalence_class(int_mod_rel(n), a))
    Option.some(z) = Option.some(za)
    some_injective[Zmod[n]](z, za)
    z = za
    za = z
    exists(a0: Int) {
        zmod_mk(n, a0) = z
    }
}

theorem zmod_mk_eq_of_rel(n: Nat, a: Int, b: Int) {
    int_mod_rel(n, a, b) implies zmod_mk(n, a) = zmod_mk(n, b)
} by {
    if int_mod_rel(n, a, b) {
        let za = zmod_mk(n, a)
        let zb = zmod_mk(n, b)
        int_mod_rel_is_equivalence(n)
        equivalence_class_eq_of_equiv(int_mod_rel(n), a, b)
        equivalence_class(int_mod_rel(n), a) = equivalence_class(int_mod_rel(n), b)
        Zmod[n].new(equivalence_class(int_mod_rel(n), a)) = Option.some(za)
        Zmod[n].new(equivalence_class(int_mod_rel(n), b)) = Option.some(zb)
        Zmod[n].new(equivalence_class(int_mod_rel(n), a)) =
            Zmod[n].new(equivalence_class(int_mod_rel(n), b))
        Option.some(za) = Option.some(zb)
        some_injective[Zmod[n]](za, zb)
        za = zb
        zmod_mk(n, a) = zmod_mk(n, b)
    }
}

theorem zmod_mk_mul_compatible(n: Nat, a1: Int, a2: Int, b1: Int, b2: Int) {
    int_mod_rel(n, a1, a2) and int_mod_rel(n, b1, b2) implies
    zmod_mk(n, a1 * b1) = zmod_mk(n, a2 * b2)
} by {
    if int_mod_rel(n, a1, a2) and int_mod_rel(n, b1, b2) {
        int_mod_rel_mul_compatible(n, a1, a2, b1, b2)
        int_mod_rel(n, a1 * b1, a2 * b2)
        zmod_mk_eq_of_rel(n, a1 * b1, a2 * b2)
    }
}

theorem zmod_mk_add_compatible(n: Nat, a1: Int, a2: Int, b1: Int, b2: Int) {
    int_mod_rel(n, a1, a2) and int_mod_rel(n, b1, b2) implies
    zmod_mk(n, a1 + b1) = zmod_mk(n, a2 + b2)
} by {
    if int_mod_rel(n, a1, a2) and int_mod_rel(n, b1, b2) {
        int_mod_rel_add_compatible(n, a1, a2, b1, b2)
        int_mod_rel(n, a1 + b1, a2 + b2)
        zmod_mk_eq_of_rel(n, a1 + b1, a2 + b2)
    }
}

/// The class produced by `zmod_mk` has the expected carrier.
theorem zmod_mk_carrier(n: Nat, a: Int) {
    zmod_mk(n, a).carrier = equivalence_class(int_mod_rel(n), a)
} by {
    mk_new_self(n, a)
    new_round_trip(n, equivalence_class(int_mod_rel(n), a), zmod_mk(n, a))
}

/// Equality of residue classes forces congruence of representatives.
theorem zmod_rel_of_mk_eq(n: Nat, a: Int, b: Int) {
    zmod_mk(n, a) = zmod_mk(n, b) implies int_mod_rel(n, a, b)
} by {
    if zmod_mk(n, a) = zmod_mk(n, b) {
        zmod_mk_carrier(n, a)
        zmod_mk_carrier(n, b)
        equivalence_class(int_mod_rel(n), a) = equivalence_class(int_mod_rel(n), b)
        int_mod_rel_is_equivalence(n)
        equivalence_of_equivalence_class_eq(int_mod_rel(n), a, b)
    }
}

/// Negation of congruent representatives gives the same residue class.
theorem zmod_mk_neg_compatible(n: Nat, a: Int, b: Int) {
    int_mod_rel(n, a, b) implies zmod_mk(n, -a) = zmod_mk(n, -b)
} by {
    if int_mod_rel(n, a, b) {
        int_mod_rel_neg_compatible(n, a, b)
        int_mod_rel(n, -a, -b)
        zmod_mk_eq_of_rel(n, -a, -b)
    }
}

/// The zero residue class modulo `n`.
define zmod_zero(n: Nat) -> Zmod[n] {
    zmod_mk(n, Int.0)
}

/// The one residue class modulo `n`.
define zmod_one(n: Nat) -> Zmod[n] {
    zmod_mk(n, Int.1)
}

/// Addition of residue classes modulo `n`.
let zmod_add(n: Nat, x: Zmod[n], y: Zmod[n]) -> result: Zmod[n] satisfy {
    exists(a: Int, b: Int) {
        zmod_mk(n, a) = x and zmod_mk(n, b) = y and result = zmod_mk(n, a + b)
    }
} by {
    zmod_mk_surjective(n, x)
    let a: Int satisfy {
        zmod_mk(n, a) = x
    }
    zmod_mk_surjective(n, y)
    let b: Int satisfy {
        zmod_mk(n, b) = y
    }
    exists(a0: Int, b0: Int) {
        zmod_mk(n, a0) = x and zmod_mk(n, b0) = y and
        zmod_mk(n, a + b) = zmod_mk(n, a0 + b0)
    }
}

/// Additive inverse of a residue class modulo `n`.
let zmod_neg(n: Nat, x: Zmod[n]) -> result: Zmod[n] satisfy {
    exists(a: Int) {
        zmod_mk(n, a) = x and result = zmod_mk(n, -a)
    }
} by {
    zmod_mk_surjective(n, x)
    let a: Int satisfy {
        zmod_mk(n, a) = x
    }
    exists(a0: Int) {
        zmod_mk(n, a0) = x and zmod_mk(n, -a) = zmod_mk(n, -a0)
    }
}

/// Multiplication of residue classes modulo `n`.
let zmod_mul(n: Nat, x: Zmod[n], y: Zmod[n]) -> result: Zmod[n] satisfy {
    exists(a: Int, b: Int) {
        zmod_mk(n, a) = x and zmod_mk(n, b) = y and result = zmod_mk(n, a * b)
    }
} by {
    zmod_mk_surjective(n, x)
    let a: Int satisfy {
        zmod_mk(n, a) = x
    }
    zmod_mk_surjective(n, y)
    let b: Int satisfy {
        zmod_mk(n, b) = y
    }
    exists(a0: Int, b0: Int) {
        zmod_mk(n, a0) = x and zmod_mk(n, b0) = y and
        zmod_mk(n, a * b) = zmod_mk(n, a0 * b0)
    }
}

/// The zero residue class is represented by integer zero.
theorem zmod_zero_mk(n: Nat) {
    zmod_zero(n) = zmod_mk(n, Int.0)
}

/// The one residue class is represented by integer one.
theorem zmod_one_mk(n: Nat) {
    zmod_one(n) = zmod_mk(n, Int.1)
}

/// Addition of residue classes agrees with addition of representatives.
theorem zmod_add_mk(n: Nat, a: Int, b: Int) {
    zmod_add(n, zmod_mk(n, a), zmod_mk(n, b)) = zmod_mk(n, a + b)
} by {
    let (x: Int, y: Int) satisfy {
        zmod_mk(n, x) = zmod_mk(n, a) and zmod_mk(n, y) = zmod_mk(n, b) and
        zmod_add(n, zmod_mk(n, a), zmod_mk(n, b)) = zmod_mk(n, x + y)
    }
    zmod_rel_of_mk_eq(n, x, a)
    zmod_rel_of_mk_eq(n, y, b)
    int_mod_rel(n, x, a)
    int_mod_rel(n, y, b)
    zmod_mk_add_compatible(n, x, a, y, b)
    zmod_mk(n, x + y) = zmod_mk(n, a + b)
}

/// Negation of a residue class agrees with negation of representatives.
theorem zmod_neg_mk(n: Nat, a: Int) {
    zmod_neg(n, zmod_mk(n, a)) = zmod_mk(n, -a)
} by {
    let x: Int satisfy {
        zmod_mk(n, x) = zmod_mk(n, a) and zmod_neg(n, zmod_mk(n, a)) = zmod_mk(n, -x)
    }
    zmod_rel_of_mk_eq(n, x, a)
    int_mod_rel(n, x, a)
    zmod_mk_neg_compatible(n, x, a)
    zmod_mk(n, -x) = zmod_mk(n, -a)
}

/// Multiplication of residue classes agrees with multiplication of representatives.
theorem zmod_mul_mk(n: Nat, a: Int, b: Int) {
    zmod_mul(n, zmod_mk(n, a), zmod_mk(n, b)) = zmod_mk(n, a * b)
} by {
    let (x: Int, y: Int) satisfy {
        zmod_mk(n, x) = zmod_mk(n, a) and zmod_mk(n, y) = zmod_mk(n, b) and
        zmod_mul(n, zmod_mk(n, a), zmod_mk(n, b)) = zmod_mk(n, x * y)
    }
    zmod_rel_of_mk_eq(n, x, a)
    zmod_rel_of_mk_eq(n, y, b)
    int_mod_rel(n, x, a)
    int_mod_rel(n, y, b)
    zmod_mk_mul_compatible(n, x, a, y, b)
    zmod_mk(n, x * y) = zmod_mk(n, a * b)
}

attributes Zmod[n: Nat] {
    /// The zero residue class.
    let zero: Zmod[n] = zmod_zero(n)

    /// The one residue class.
    let one: Zmod[n] = zmod_one(n)

    /// Addition of residue classes.
    define add(self, other: Zmod[n]) -> Zmod[n] {
        zmod_add(n, self, other)
    }

    /// Additive inverse of a residue class.
    define neg(self) -> Zmod[n] {
        zmod_neg(n, self)
    }

    /// Multiplication of residue classes.
    define mul(self, other: Zmod[n]) -> Zmod[n] {
        zmod_mul(n, self, other)
    }
}

/// Residue classes have addition.
instance Zmod[n: Nat]: Add {
    let add = Zmod[n].add
}

/// Residue classes have a zero element.
instance Zmod[n: Nat]: Zero {
    let 0 = Zmod[n].zero
}

/// Residue classes have additive inverses.
instance Zmod[n: Nat]: Neg {
    let neg = Zmod[n].neg
}

/// Residue classes have multiplication.
instance Zmod[n: Nat]: Mul {
    let mul = Zmod[n].mul
}

/// Residue classes have a one element.
instance Zmod[n: Nat]: One {
    let 1 = Zmod[n].one
}

/// Addition of residue classes is associative.
theorem zmod_add_assoc(n: Nat, x: Zmod[n], y: Zmod[n], z: Zmod[n]) {
    zmod_add(n, zmod_add(n, x, y), z) = zmod_add(n, x, zmod_add(n, y, z))
} by {
    zmod_mk_surjective(n, x)
    let a: Int satisfy { zmod_mk(n, a) = x }
    zmod_mk_surjective(n, y)
    let b: Int satisfy { zmod_mk(n, b) = y }
    zmod_mk_surjective(n, z)
    let c: Int satisfy { zmod_mk(n, c) = z }
    zmod_add_mk(n, a, b)
    zmod_add_mk(n, a + b, c)
    zmod_add_mk(n, b, c)
    zmod_add_mk(n, a, b + c)
    add_assoc(a, b, c)
}

/// Addition of residue classes is commutative.
theorem zmod_add_comm(n: Nat, x: Zmod[n], y: Zmod[n]) {
    zmod_add(n, x, y) = zmod_add(n, y, x)
} by {
    zmod_mk_surjective(n, x)
    let a: Int satisfy { zmod_mk(n, a) = x }
    zmod_mk_surjective(n, y)
    let b: Int satisfy { zmod_mk(n, b) = y }
    zmod_add_mk(n, a, b)
    zmod_add_mk(n, b, a)
    add_comm(a, b)
}

/// Zero is a left identity for addition of residue classes.
theorem zmod_add_zero_left(n: Nat, x: Zmod[n]) {
    zmod_add(n, zmod_zero(n), x) = x
} by {
    zmod_mk_surjective(n, x)
    let a: Int satisfy { zmod_mk(n, a) = x }
    zmod_zero_mk(n)
    zmod_add_mk(n, Int.0, a)
    add_zero_left(a)
}

/// Zero is a right identity for addition of residue classes.
theorem zmod_add_zero_right(n: Nat, x: Zmod[n]) {
    zmod_add(n, x, zmod_zero(n)) = x
} by {
    zmod_mk_surjective(n, x)
    let a: Int satisfy { zmod_mk(n, a) = x }
    zmod_zero_mk(n)
    zmod_add_mk(n, a, Int.0)
    add_zero_right(a)
}

/// Negation is a left additive inverse for residue classes.
theorem zmod_add_neg_left(n: Nat, x: Zmod[n]) {
    zmod_add(n, zmod_neg(n, x), x) = zmod_zero(n)
} by {
    zmod_mk_surjective(n, x)
    let a: Int satisfy { zmod_mk(n, a) = x }
    zmod_neg_mk(n, a)
    zmod_add_mk(n, -a, a)
    zmod_zero_mk(n)
    add_neg(a)
    add_comm(-a, a)
}

/// Negation is a right additive inverse for residue classes.
theorem zmod_add_neg_right(n: Nat, x: Zmod[n]) {
    zmod_add(n, x, zmod_neg(n, x)) = zmod_zero(n)
} by {
    zmod_mk_surjective(n, x)
    let a: Int satisfy { zmod_mk(n, a) = x }
    zmod_neg_mk(n, a)
    zmod_add_mk(n, a, -a)
    zmod_zero_mk(n)
    add_neg(a)
}

/// Multiplication of residue classes is associative.
theorem zmod_mul_assoc(n: Nat, x: Zmod[n], y: Zmod[n], z: Zmod[n]) {
    zmod_mul(n, zmod_mul(n, x, y), z) = zmod_mul(n, x, zmod_mul(n, y, z))
} by {
    zmod_mk_surjective(n, x)
    let a: Int satisfy { zmod_mk(n, a) = x }
    zmod_mk_surjective(n, y)
    let b: Int satisfy { zmod_mk(n, b) = y }
    zmod_mk_surjective(n, z)
    let c: Int satisfy { zmod_mk(n, c) = z }
    zmod_mul_mk(n, a, b)
    zmod_mul_mk(n, a * b, c)
    zmod_mul_mk(n, b, c)
    zmod_mul_mk(n, a, b * c)
    mul_assoc(a, b, c)
}

/// Multiplication of residue classes is commutative.
theorem zmod_mul_comm(n: Nat, x: Zmod[n], y: Zmod[n]) {
    zmod_mul(n, x, y) = zmod_mul(n, y, x)
} by {
    zmod_mk_surjective(n, x)
    let a: Int satisfy { zmod_mk(n, a) = x }
    zmod_mk_surjective(n, y)
    let b: Int satisfy { zmod_mk(n, b) = y }
    zmod_mul_mk(n, a, b)
    zmod_mul_mk(n, b, a)
    mul_comm(a, b)
}

/// One is a left identity for multiplication of residue classes.
theorem zmod_mul_one_left(n: Nat, x: Zmod[n]) {
    zmod_mul(n, zmod_one(n), x) = x
} by {
    zmod_mk_surjective(n, x)
    let a: Int satisfy { zmod_mk(n, a) = x }
    zmod_one_mk(n)
    zmod_mul_mk(n, Int.1, a)
    mul_one_left(a)
}

/// One is a right identity for multiplication of residue classes.
theorem zmod_mul_one_right(n: Nat, x: Zmod[n]) {
    zmod_mul(n, x, zmod_one(n)) = x
} by {
    zmod_mk_surjective(n, x)
    let a: Int satisfy { zmod_mk(n, a) = x }
    zmod_one_mk(n)
    zmod_mul_mk(n, a, Int.1)
    mul_one_right(a)
}

/// Zero is a left annihilator for multiplication of residue classes.
theorem zmod_mul_zero_left(n: Nat, x: Zmod[n]) {
    zmod_mul(n, zmod_zero(n), x) = zmod_zero(n)
} by {
    zmod_mk_surjective(n, x)
    let a: Int satisfy { zmod_mk(n, a) = x }
    zmod_zero_mk(n)
    zmod_mul_mk(n, Int.0, a)
    mul_zero_left(a)
}

/// Zero is a right annihilator for multiplication of residue classes.
theorem zmod_mul_zero_right(n: Nat, x: Zmod[n]) {
    zmod_mul(n, x, zmod_zero(n)) = zmod_zero(n)
} by {
    zmod_mk_surjective(n, x)
    let a: Int satisfy { zmod_mk(n, a) = x }
    zmod_zero_mk(n)
    zmod_mul_mk(n, a, Int.0)
    mul_zero_right(a)
}

/// Multiplication distributes on the left over addition of residue classes.
theorem zmod_mul_add_left(n: Nat, x: Zmod[n], y: Zmod[n], z: Zmod[n]) {
    zmod_mul(n, x, zmod_add(n, y, z)) =
        zmod_add(n, zmod_mul(n, x, y), zmod_mul(n, x, z))
} by {
    zmod_mk_surjective(n, x)
    let a: Int satisfy { zmod_mk(n, a) = x }
    zmod_mk_surjective(n, y)
    let b: Int satisfy { zmod_mk(n, b) = y }
    zmod_mk_surjective(n, z)
    let c: Int satisfy { zmod_mk(n, c) = z }
    zmod_add_mk(n, b, c)
    zmod_mul_mk(n, a, b + c)
    zmod_mul_mk(n, a, b)
    zmod_mul_mk(n, a, c)
    zmod_add_mk(n, a * b, a * c)
    mul_distrib_left(a, b, c)
}

/// Multiplication distributes on the right over addition of residue classes.
theorem zmod_mul_add_right(n: Nat, x: Zmod[n], y: Zmod[n], z: Zmod[n]) {
    zmod_mul(n, zmod_add(n, x, y), z) =
        zmod_add(n, zmod_mul(n, x, z), zmod_mul(n, y, z))
} by {
    zmod_mk_surjective(n, x)
    let a: Int satisfy { zmod_mk(n, a) = x }
    zmod_mk_surjective(n, y)
    let b: Int satisfy { zmod_mk(n, b) = y }
    zmod_mk_surjective(n, z)
    let c: Int satisfy { zmod_mk(n, c) = z }
    zmod_add_mk(n, a, b)
    zmod_mul_mk(n, a + b, c)
    zmod_mul_mk(n, a, c)
    zmod_mul_mk(n, b, c)
    zmod_add_mk(n, a * c, b * c)
    mul_distrib_right(a, b, c)
}

theorem attribute_round_trip(n: Nat, x: Zmod[n]) {
    x.modulus = n and x.again = x
}
