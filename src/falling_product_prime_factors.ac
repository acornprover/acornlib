from nat import Nat, divides_self, divides_mul, divides_trans, lte_and_lt, lt_and_lte,
    lt_imp_lte_suc, lte_cancel_suc, lte_antisymm, lt_or_lte, only_zero_lte_zero,
    lte_trans
from number_theory import falling_product, falling_product_zero, falling_product_suc,
    prime_divides_mul

numerals Nat

/// The whole falling product is divisible by any one of its factors.
///
/// The factors are `n, n - 1, ..., n - k`, and the product is built one factor at a time, so
/// each is picked up either at the last step or by the shorter product inside it.
theorem falling_product_factor_divides(n: Nat, k: Nat, i: Nat) {
    i <= k implies (n - i).divides(falling_product(n, k))
} by {
    define p(x: Nat) -> Bool {
        forall(j: Nat) {
            j <= x implies (n - j).divides(falling_product(n, x))
        }
    }
    forall(j: Nat) {
        if j <= Nat.0 {
            only_zero_lte_zero(j)
            j = Nat.0
            falling_product_zero(n)
            falling_product(n, Nat.0) = n
            n - Nat.0 = n
            divides_self(n)
            (n - j).divides(falling_product(n, Nat.0))
        }
        (j <= Nat.0 implies (n - j).divides(falling_product(n, Nat.0)))
    }
    p(Nat.0)
    forall(m: Nat) {
        if p(m) {
            forall(j: Nat) {
                if j <= m.suc {
                    falling_product_suc(n, m)
                    falling_product(n, m.suc) = falling_product(n, m) * (n - m.suc)
                    if j = m.suc {
                        divides_self(n - m.suc)
                        (n - m.suc).divides(falling_product(n, m) * (n - m.suc))
                        (n - j).divides(falling_product(n, m.suc))
                    }
                    if j != m.suc {
                        lt_imp_lte_suc(j, m.suc)
                        j.suc <= m.suc
                        lte_cancel_suc(j, m)
                        j <= m
                        (j <= m implies (n - j).divides(falling_product(n, m)))
                        (n - j).divides(falling_product(n, m))
                        divides_mul(falling_product(n, m), n - m.suc, n - j)
                        (n - j).divides(falling_product(n, m) * (n - m.suc))
                        (n - j).divides(falling_product(n, m.suc))
                    }
                    (n - j).divides(falling_product(n, m.suc))
                }
                (j <= m.suc implies (n - j).divides(falling_product(n, m.suc)))
            }
            p(m.suc)
        }
        (p(m) implies p(m.suc))
    }
    p(Nat.0) and forall(m: Nat) {
        p(m) implies p(m.suc)
    }
    Nat.induction(p)
    p(k)
    (i <= k implies (n - i).divides(falling_product(n, k)))
}

/// A prime dividing a falling product divides one of its factors.
///
/// Euclid's lemma applied along the product. This is the direction that was missing: the
/// valuation form `count_prime_factor_falling_product` adds up multiplicities, but says
/// nothing about which factor a given prime came from.
theorem prime_divides_falling_product_factor(n: Nat, k: Nat, q: Nat) {
    q.is_prime and q.divides(falling_product(n, k))
        implies exists(i: Nat) { i <= k and q.divides(n - i) }
} by {
    if q.is_prime {
        define p(x: Nat) -> Bool {
            q.divides(falling_product(n, x))
                implies exists(i: Nat) { i <= x and q.divides(n - i) }
        }
        if q.divides(falling_product(n, Nat.0)) {
            falling_product_zero(n)
            falling_product(n, Nat.0) = n
            q.divides(n)
            n - Nat.0 = n
            q.divides(n - Nat.0)
            Nat.0 <= Nat.0
            exists(i: Nat) { i <= Nat.0 and q.divides(n - i) }
        }
        p(Nat.0)
        forall(m: Nat) {
            if p(m) {
                if q.divides(falling_product(n, m.suc)) {
                    falling_product_suc(n, m)
                    falling_product(n, m.suc) = falling_product(n, m) * (n - m.suc)
                    q.divides(falling_product(n, m) * (n - m.suc))
                    prime_divides_mul(q, falling_product(n, m), n - m.suc)
                    (q.divides(falling_product(n, m)) or q.divides(n - m.suc))
                    if q.divides(falling_product(n, m)) {
                        (q.divides(falling_product(n, m))
                            implies exists(i: Nat) { i <= m and q.divides(n - i) })
                        exists(i: Nat) { i <= m and q.divides(n - i) }
                        let (i: Nat) satisfy {
                            i <= m and q.divides(n - i)
                        }
                        m <= m.suc
                        lte_trans(i, m, m.suc)
                        i <= m.suc
                        exists(j: Nat) { j <= m.suc and q.divides(n - j) }
                    }
                    if not q.divides(falling_product(n, m)) {
                        q.divides(n - m.suc)
                        m.suc <= m.suc
                        exists(j: Nat) { j <= m.suc and q.divides(n - j) }
                    }
                    exists(j: Nat) { j <= m.suc and q.divides(n - j) }
                }
                p(m.suc)
            }
            (p(m) implies p(m.suc))
        }
        p(Nat.0) and forall(m: Nat) {
            p(m) implies p(m.suc)
        }
        Nat.induction(p)
        p(k)
        (q.divides(falling_product(n, k))
            implies exists(i: Nat) { i <= k and q.divides(n - i) })
    }
}

/// A prime dividing one of the factors divides the falling product.
theorem prime_factor_divides_falling_product(n: Nat, k: Nat, q: Nat, i: Nat) {
    i <= k and q.divides(n - i) implies q.divides(falling_product(n, k))
} by {
    if i <= k and q.divides(n - i) {
        falling_product_factor_divides(n, k, i)
        (n - i).divides(falling_product(n, k))
        divides_trans(q, n - i, falling_product(n, k))
        q.divides(falling_product(n, k))
    }
}

/// The prime factors of a falling product are exactly those of its factors.
///
/// Both directions together: a prime divides the product precisely when it divides one of the
/// terms `n, n - 1, ..., n - k`.
theorem prime_divides_falling_product_iff(n: Nat, k: Nat, q: Nat) {
    q.is_prime implies (q.divides(falling_product(n, k))
        = exists(i: Nat) { i <= k and q.divides(n - i) })
} by {
    if q.is_prime {
        if q.divides(falling_product(n, k)) {
            prime_divides_falling_product_factor(n, k, q)
            exists(i: Nat) { i <= k and q.divides(n - i) }
        }
        if exists(i: Nat) { i <= k and q.divides(n - i) } {
            let (i: Nat) satisfy {
                i <= k and q.divides(n - i)
            }
            prime_factor_divides_falling_product(n, k, q, i)
            q.divides(falling_product(n, k))
        }
        (q.divides(falling_product(n, k))
            implies exists(i: Nat) { i <= k and q.divides(n - i) })
        ((exists(i: Nat) { i <= k and q.divides(n - i) })
            implies q.divides(falling_product(n, k)))
        (q.divides(falling_product(n, k))
            = exists(i: Nat) { i <= k and q.divides(n - i) })
    }
}
