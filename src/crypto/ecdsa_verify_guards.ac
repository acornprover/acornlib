/// Guard predicates and API bridges for ECDSA signature verification.

from crypto.ecdsa import XCoord, is_mod_inv, ecdsa_verify
from nat import Nat, not_lt_zero

numerals Nat

/// The public range check on the ECDSA `r` scalar.
define ecdsa_r_in_range(n: Nat, r: Nat) -> Bool {
    0 < r and r < n
}

/// The public range check on the ECDSA `s` scalar.
define ecdsa_s_in_range(n: Nat, s: Nat) -> Bool {
    0 < s and s < n
}

/// The public range checks on the two ECDSA signature scalars.
define ecdsa_signature_components_in_range(n: Nat, r: Nat, s: Nat) -> Bool {
    ecdsa_r_in_range(n, r) and ecdsa_s_in_range(n, s)
}

/// The non-curve arithmetic guards checked before the ECDSA verifier equation.
define ecdsa_verify_public_guards(n: Nat, r: Nat, s: Nat, sinv: Nat) -> Bool {
    ecdsa_signature_components_in_range(n, r, s) and is_mod_inv(s, sinv, n)
}

/// The ECDSA verifier's final group-expression congruence.
define ecdsa_verify_core_equation[G: XCoord](
    n: Nat,
    gen: G,
    pub: G,
    h: Nat,
    r: Nat,
    sinv: Nat
) -> Bool {
    (gen.pow((h * sinv).mod(n)) * pub.pow((r * sinv).mod(n))).x_coord.mod(n) = r
}

/// The ECDSA `r` range guard gives the upper bound on `r`.
theorem ecdsa_r_in_range_lt(n: Nat, r: Nat) {
    ecdsa_r_in_range(n, r) implies r < n
} by {
    if ecdsa_r_in_range(n, r) {
        ecdsa_r_in_range(n, r) = (0 < r and r < n)
        r < n
    }
}

/// The ECDSA `s` range guard gives the upper bound on `s`.
theorem ecdsa_s_in_range_lt(n: Nat, s: Nat) {
    ecdsa_s_in_range(n, s) implies s < n
} by {
    if ecdsa_s_in_range(n, s) {
        ecdsa_s_in_range(n, s) = (0 < s and s < n)
        s < n
    }
}

/// The combined range guard gives the `r` range guard.
theorem ecdsa_signature_components_in_range_r(n: Nat, r: Nat, s: Nat) {
    ecdsa_signature_components_in_range(n, r, s) implies ecdsa_r_in_range(n, r)
} by {
    if ecdsa_signature_components_in_range(n, r, s) {
        ecdsa_signature_components_in_range(n, r, s) =
            (ecdsa_r_in_range(n, r) and ecdsa_s_in_range(n, s))
        ecdsa_r_in_range(n, r)
    }
}

/// The combined range guard gives the `s` range guard.
theorem ecdsa_signature_components_in_range_s(n: Nat, r: Nat, s: Nat) {
    ecdsa_signature_components_in_range(n, r, s) implies ecdsa_s_in_range(n, s)
} by {
    if ecdsa_signature_components_in_range(n, r, s) {
        ecdsa_signature_components_in_range(n, r, s) =
            (ecdsa_r_in_range(n, r) and ecdsa_s_in_range(n, s))
        ecdsa_s_in_range(n, s)
    }
}

/// The combined range guard gives the upper bound on `r`.
theorem ecdsa_signature_components_in_range_r_lt(n: Nat, r: Nat, s: Nat) {
    ecdsa_signature_components_in_range(n, r, s) implies r < n
} by {
    if ecdsa_signature_components_in_range(n, r, s) {
        ecdsa_signature_components_in_range_r(n, r, s)
        ecdsa_r_in_range(n, r)
        ecdsa_r_in_range_lt(n, r)
        r < n
    }
}

/// The combined range guard gives the upper bound on `s`.
theorem ecdsa_signature_components_in_range_s_lt(n: Nat, r: Nat, s: Nat) {
    ecdsa_signature_components_in_range(n, r, s) implies s < n
} by {
    if ecdsa_signature_components_in_range(n, r, s) {
        ecdsa_signature_components_in_range_s(n, r, s)
        ecdsa_s_in_range(n, s)
        ecdsa_s_in_range_lt(n, s)
        s < n
    }
}

/// The public verifier guards include the signature component range guards.
theorem ecdsa_verify_public_guards_components(n: Nat, r: Nat, s: Nat, sinv: Nat) {
    ecdsa_verify_public_guards(n, r, s, sinv) implies
        ecdsa_signature_components_in_range(n, r, s)
} by {
    if ecdsa_verify_public_guards(n, r, s, sinv) {
        ecdsa_verify_public_guards(n, r, s, sinv) =
            (ecdsa_signature_components_in_range(n, r, s) and is_mod_inv(s, sinv, n))
        ecdsa_signature_components_in_range(n, r, s)
    }
}

/// The public verifier guards include the modular-inverse guard for `s`.
theorem ecdsa_verify_public_guards_inverse(n: Nat, r: Nat, s: Nat, sinv: Nat) {
    ecdsa_verify_public_guards(n, r, s, sinv) implies is_mod_inv(s, sinv, n)
} by {
    if ecdsa_verify_public_guards(n, r, s, sinv) {
        ecdsa_verify_public_guards(n, r, s, sinv) =
            (ecdsa_signature_components_in_range(n, r, s) and is_mod_inv(s, sinv, n))
        is_mod_inv(s, sinv, n)
    }
}

/// Verification implies the public range and inverse guards.
theorem ecdsa_verify_public_guards_of_verify[G: XCoord](
    n: Nat,
    gen: G,
    pub: G,
    h: Nat,
    r: Nat,
    s: Nat,
    sinv: Nat
) {
    ecdsa_verify(n, gen, pub, h, r, s, sinv) implies ecdsa_verify_public_guards(n, r, s, sinv)
} by {
    if ecdsa_verify(n, gen, pub, h, r, s, sinv) {
        ecdsa_verify(n, gen, pub, h, r, s, sinv) =
            (0 < r and r < n and 0 < s and s < n and
             is_mod_inv(s, sinv, n) and
             (gen.pow((h * sinv).mod(n)) * pub.pow((r * sinv).mod(n))).x_coord.mod(n) = r)
        ecdsa_signature_components_in_range(n, r, s)
        ecdsa_verify_public_guards(n, r, s, sinv)
    }
}

/// Verification implies the `r` range guard.
theorem ecdsa_verify_r_in_range_of_verify[G: XCoord](
    n: Nat,
    gen: G,
    pub: G,
    h: Nat,
    r: Nat,
    s: Nat,
    sinv: Nat
) {
    ecdsa_verify(n, gen, pub, h, r, s, sinv) implies ecdsa_r_in_range(n, r)
} by {
    if ecdsa_verify(n, gen, pub, h, r, s, sinv) {
        ecdsa_verify_public_guards_of_verify(n, gen, pub, h, r, s, sinv)
        ecdsa_verify_public_guards(n, r, s, sinv)
        ecdsa_verify_public_guards_components(n, r, s, sinv)
        ecdsa_signature_components_in_range(n, r, s)
        ecdsa_signature_components_in_range_r(n, r, s)
        ecdsa_r_in_range(n, r)
    }
}

/// Verification implies the `s` range guard.
theorem ecdsa_verify_s_in_range_of_verify[G: XCoord](
    n: Nat,
    gen: G,
    pub: G,
    h: Nat,
    r: Nat,
    s: Nat,
    sinv: Nat
) {
    ecdsa_verify(n, gen, pub, h, r, s, sinv) implies ecdsa_s_in_range(n, s)
} by {
    if ecdsa_verify(n, gen, pub, h, r, s, sinv) {
        ecdsa_verify_public_guards_of_verify(n, gen, pub, h, r, s, sinv)
        ecdsa_verify_public_guards(n, r, s, sinv)
        ecdsa_verify_public_guards_components(n, r, s, sinv)
        ecdsa_signature_components_in_range(n, r, s)
        ecdsa_signature_components_in_range_s(n, r, s)
        ecdsa_s_in_range(n, s)
    }
}

/// Verification implies the modular-inverse guard for `s`.
theorem ecdsa_verify_inverse_of_verify[G: XCoord](
    n: Nat,
    gen: G,
    pub: G,
    h: Nat,
    r: Nat,
    s: Nat,
    sinv: Nat
) {
    ecdsa_verify(n, gen, pub, h, r, s, sinv) implies is_mod_inv(s, sinv, n)
} by {
    if ecdsa_verify(n, gen, pub, h, r, s, sinv) {
        ecdsa_verify_public_guards_of_verify(n, gen, pub, h, r, s, sinv)
        ecdsa_verify_public_guards(n, r, s, sinv)
        ecdsa_verify_public_guards_inverse(n, r, s, sinv)
        is_mod_inv(s, sinv, n)
    }
}

/// Verification implies the final verifier equation.
theorem ecdsa_verify_core_equation_of_verify[G: XCoord](
    n: Nat,
    gen: G,
    pub: G,
    h: Nat,
    r: Nat,
    s: Nat,
    sinv: Nat
) {
    ecdsa_verify(n, gen, pub, h, r, s, sinv) implies
        ecdsa_verify_core_equation(n, gen, pub, h, r, sinv)
} by {
    if ecdsa_verify(n, gen, pub, h, r, s, sinv) {
        ecdsa_verify(n, gen, pub, h, r, s, sinv) =
            (0 < r and r < n and 0 < s and s < n and
             is_mod_inv(s, sinv, n) and
             (gen.pow((h * sinv).mod(n)) * pub.pow((r * sinv).mod(n))).x_coord.mod(n) = r)
        ecdsa_verify_core_equation(n, gen, pub, h, r, sinv)
    }
}

/// The public guards imply that the verification modulus is nonzero.
theorem ecdsa_verify_public_guards_modulus_ne_zero(n: Nat, r: Nat, s: Nat, sinv: Nat) {
    ecdsa_verify_public_guards(n, r, s, sinv) implies n != 0
} by {
    if ecdsa_verify_public_guards(n, r, s, sinv) {
        ecdsa_verify_public_guards_components(n, r, s, sinv)
        ecdsa_signature_components_in_range(n, r, s)
        ecdsa_signature_components_in_range_r_lt(n, r, s)
        r < n
        if n = 0 {
            r < 0
            not_lt_zero(r)
            false
        }
    }
}

/// Verification implies that the verification modulus is nonzero.
theorem ecdsa_verify_modulus_ne_zero[G: XCoord](
    n: Nat,
    gen: G,
    pub: G,
    h: Nat,
    r: Nat,
    s: Nat,
    sinv: Nat
) {
    ecdsa_verify(n, gen, pub, h, r, s, sinv) implies n != 0
} by {
    if ecdsa_verify(n, gen, pub, h, r, s, sinv) {
        ecdsa_verify_public_guards_of_verify(n, gen, pub, h, r, s, sinv)
        ecdsa_verify_public_guards(n, r, s, sinv)
        ecdsa_verify_public_guards_modulus_ne_zero(n, r, s, sinv)
        n != 0
    }
}

/// Public guards together with the verifier equation reconstruct `ecdsa_verify`.
theorem ecdsa_verify_of_public_guards_and_core_equation[G: XCoord](
    n: Nat,
    gen: G,
    pub: G,
    h: Nat,
    r: Nat,
    s: Nat,
    sinv: Nat
) {
    ecdsa_verify_public_guards(n, r, s, sinv) and
        ecdsa_verify_core_equation(n, gen, pub, h, r, sinv)
    implies ecdsa_verify(n, gen, pub, h, r, s, sinv)
} by {
    if ecdsa_verify_public_guards(n, r, s, sinv) and
        ecdsa_verify_core_equation(n, gen, pub, h, r, sinv) {
        ecdsa_verify_public_guards(n, r, s, sinv) =
            (ecdsa_signature_components_in_range(n, r, s) and is_mod_inv(s, sinv, n))
        ecdsa_signature_components_in_range(n, r, s) =
            (ecdsa_r_in_range(n, r) and ecdsa_s_in_range(n, s))
        ecdsa_r_in_range(n, r) = (0 < r and r < n)
        ecdsa_s_in_range(n, s) = (0 < s and s < n)
        ecdsa_verify(n, gen, pub, h, r, s, sinv)
    }
}
