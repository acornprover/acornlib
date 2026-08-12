from crypto.rsa import Nat, rsa_encrypt, rsa_decrypt
from nat import mod_by_zero, mod_mod
from number_theory import mod_lt

numerals Nat

/// RSA encryption output is below a nonzero modulus.
theorem rsa_encrypt_output_lt(m: Nat, e: Nat, n: Nat) {
    n != Nat.0 implies rsa_encrypt(m, e, n) < n
} by {
    if n != Nat.0 {
        rsa_encrypt(m, e, n) = m.pow(e).mod(n)
        mod_lt(m.pow(e), n)
    }
}

/// RSA decryption output is below a nonzero modulus.
theorem rsa_decrypt_output_lt(c: Nat, d: Nat, n: Nat) {
    n != Nat.0 implies rsa_decrypt(c, d, n) < n
} by {
    if n != Nat.0 {
        rsa_decrypt(c, d, n) = c.pow(d).mod(n)
        mod_lt(c.pow(d), n)
    }
}

/// RSA encryption output is already reduced modulo the modulus.
theorem rsa_encrypt_mod_self(m: Nat, e: Nat, n: Nat) {
    rsa_encrypt(m, e, n).mod(n) = rsa_encrypt(m, e, n)
} by {
    rsa_encrypt(m, e, n) = m.pow(e).mod(n)
    mod_mod(m.pow(e), n)
}

/// RSA decryption output is already reduced modulo the modulus.
theorem rsa_decrypt_mod_self(c: Nat, d: Nat, n: Nat) {
    rsa_decrypt(c, d, n).mod(n) = rsa_decrypt(c, d, n)
} by {
    rsa_decrypt(c, d, n) = c.pow(d).mod(n)
    mod_mod(c.pow(d), n)
}

/// RSA encryption with modulus zero is raw exponentiation.
theorem rsa_encrypt_zero_modulus(m: Nat, e: Nat) {
    rsa_encrypt(m, e, Nat.0) = m.pow(e)
} by {
    rsa_encrypt(m, e, Nat.0) = m.pow(e).mod(Nat.0)
    mod_by_zero(m.pow(e))
}

/// RSA decryption with modulus zero is raw exponentiation.
theorem rsa_decrypt_zero_modulus(c: Nat, d: Nat) {
    rsa_decrypt(c, d, Nat.0) = c.pow(d)
} by {
    rsa_decrypt(c, d, Nat.0) = c.pow(d).mod(Nat.0)
    mod_by_zero(c.pow(d))
}
