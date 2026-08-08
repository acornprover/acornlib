from algebra.group import Group
from nat import Nat

numerals Nat

/// RSA encryption of a message `m` with public exponent `e` modulo `n`.
/// Computes `m^e mod n`.
define rsa_encrypt(m: Nat, e: Nat, n: Nat) -> Nat {
    m.pow(e).mod(n)
}

/// RSA decryption of a ciphertext `c` with private exponent `d` modulo `n`.
/// Computes `c^d mod n`.
define rsa_decrypt(c: Nat, d: Nat, n: Nat) -> Nat {
    c.pow(d).mod(n)
}

/// True if `(p, q, e, d)` is a valid RSA key tuple: `p` and `q` are distinct
/// primes and `e * d` is congruent to `1` modulo `(p - 1) * (q - 1)`.
define is_rsa_key(p: Nat, q: Nat, e: Nat, d: Nat) -> Bool {
    p.is_prime and q.is_prime and p != q and
    (e * d).mod((p - 1) * (q - 1)) = 1
}

/// Bridge: is_rsa_key unfolds to its prime/distinctness/exponent clauses.
theorem is_rsa_key_unfold(p: Nat, q: Nat, e: Nat, d: Nat) {
    is_rsa_key(p, q, e, d) =
        (p.is_prime and q.is_prime and p != q
            and (e * d).mod((p - 1) * (q - 1)) = 1)
}

/// RSA correctness: decrypting an encrypted message recovers the original.
/// For a valid RSA key `(p, q, e, d)` with modulus `n = p * q` and any
/// message `m < n`, we have `rsa_decrypt(rsa_encrypt(m, e, n), d, n) = m`.
theorem rsa_correctness(p: Nat, q: Nat, e: Nat, d: Nat, m: Nat) {
    is_rsa_key(p, q, e, d) and m < p * q implies
        rsa_decrypt(rsa_encrypt(m, e, p * q), d, p * q) = m
}

/// RSA key validity is symmetric in the public and private exponents.
theorem rsa_key_symmetric(p: Nat, q: Nat, e: Nat, d: Nat) {
    is_rsa_key(p, q, e, d) implies is_rsa_key(p, q, d, e)
}

/// Normalized RSA correctness: decrypting an encrypted message recovers the
/// message modulo the RSA modulus, with no boundedness hypothesis on `m`.
theorem rsa_decrypt_encrypt_mod(p: Nat, q: Nat, e: Nat, d: Nat, m: Nat) {
    is_rsa_key(p, q, e, d) implies
        rsa_decrypt(rsa_encrypt(m, e, p * q), d, p * q) = m.mod(p * q)
}

/// Bounded reverse RSA roundtrip: applying the private exponent and then the
/// public exponent recovers a bounded message.
theorem rsa_encrypt_decrypt_bounded(p: Nat, q: Nat, e: Nat, d: Nat, m: Nat) {
    is_rsa_key(p, q, e, d) and m < p * q implies
        rsa_encrypt(rsa_decrypt(m, d, p * q), e, p * q) = m
}

/// Normalized reverse RSA roundtrip: applying the private exponent and then
/// the public exponent recovers the input modulo the RSA modulus.
theorem rsa_encrypt_decrypt_mod(p: Nat, q: Nat, e: Nat, d: Nat, m: Nat) {
    is_rsa_key(p, q, e, d) implies
        rsa_encrypt(rsa_decrypt(m, d, p * q), e, p * q) = m.mod(p * q)
}

/// Bidirectional normalized RSA roundtrip: both public-then-private and
/// private-then-public exponentiation recover the input modulo the RSA modulus.
theorem rsa_roundtrip_mod_pair(p: Nat, q: Nat, e: Nat, d: Nat, m: Nat) {
    is_rsa_key(p, q, e, d) implies
        rsa_decrypt(rsa_encrypt(m, e, p * q), d, p * q) = m.mod(p * q) and
        rsa_encrypt(rsa_decrypt(m, d, p * q), e, p * q) = m.mod(p * q)
}

/// DSA domain parameters: a prime `p`, a prime `q` dividing `p - 1`,
/// and a generator `g` of the unique order-`q` subgroup of `(Z/p)*`.
define is_dsa_params(p: Nat, q: Nat, g: Nat) -> Bool {
    p.is_prime and q.is_prime and q.divides(p - 1) and
    1 < g and g < p and g.pow(q).mod(p) = 1
}

/// Bridge: DSA parameter validity unfolds to its arithmetic and order clauses.
theorem is_dsa_params_unfold(p: Nat, q: Nat, g: Nat) {
    is_dsa_params(p, q, g) = (p.is_prime and q.is_prime and
        q.divides(p - 1) and 1 < g and g < p and g.pow(q).mod(p) = 1)
}

/// True if `y` is the DSA public key corresponding to private key `x`
/// under parameters `(p, q, g)`.
define is_dsa_keypair(p: Nat, q: Nat, g: Nat, x: Nat, y: Nat) -> Bool {
    0 < x and x < q and y = g.pow(x).mod(p)
}

/// Bridge: DSA key-pair validity unfolds to the private-key bounds and
/// public-key exponent equation.
theorem is_dsa_keypair_unfold(p: Nat, q: Nat, g: Nat, x: Nat, y: Nat) {
    is_dsa_keypair(p, q, g, x, y) = (0 < x and x < q and y = g.pow(x).mod(p))
}

/// True if `kinv` is the modular inverse of `k` modulo `q`.
define is_mod_inverse(k: Nat, kinv: Nat, q: Nat) -> Bool {
    (k * kinv).mod(q) = 1
}

/// Bridge: DSA modular-inverse validity is the defining product congruence.
theorem is_mod_inverse_unfold(k: Nat, kinv: Nat, q: Nat) {
    is_mod_inverse(k, kinv, q) = ((k * kinv).mod(q) = 1)
}

/// The first DSA signature component: `r = (g^k mod p) mod q`.
define dsa_sign_r(p: Nat, q: Nat, g: Nat, k: Nat) -> Nat {
    g.pow(k).mod(p).mod(q)
}

/// The second DSA signature component:
/// `s = k^{-1} (h + x * r) mod q`, given a precomputed inverse `kinv` of `k`.
define dsa_sign_s(q: Nat, h: Nat, x: Nat, r: Nat, kinv: Nat) -> Nat {
    (kinv * (h + x * r)).mod(q)
}

/// The DSA `r` component is reduced modulo `q`.
theorem dsa_sign_r_lt(p: Nat, q: Nat, g: Nat, k: Nat) {
    q != 0 implies dsa_sign_r(p, q, g, k) < q
}

/// The DSA `s` component is reduced modulo `q`.
theorem dsa_sign_s_lt(q: Nat, h: Nat, x: Nat, r: Nat, kinv: Nat) {
    q != 0 implies dsa_sign_s(q, h, x, r, kinv) < q
}

/// DSA signature verification:
/// accept `(r, s)` for hash `h` under public key `y` and parameters `(p, q, g)`
/// when `r = (g^{h * w} * y^{r * w} mod p) mod q` for `w = s^{-1} mod q`.
define dsa_verify(p: Nat, q: Nat, g: Nat, y: Nat, h: Nat,
                  r: Nat, s: Nat, sinv: Nat) -> Bool {
    0 < r and r < q and 0 < s and s < q and
    is_mod_inverse(s, sinv, q) and
    (g.pow((h * sinv).mod(q)) * y.pow((r * sinv).mod(q))).mod(p).mod(q) = r
}

/// DSA verifier identity: the verifier expression collapses to `g^k mod p`
/// under the key, order, and modular-inverse hypotheses.
theorem dsa_verify_identity(p: Nat, q: Nat, g: Nat, x: Nat, y: Nat,
                             h: Nat, r: Nat, k: Nat, kinv: Nat,
                             s: Nat, sinv: Nat) {
    1 < q and
    g.pow(q).mod(p) = 1 and
    y = g.pow(x).mod(p) and
    k < q and
    (k * kinv).mod(q) = 1 and
    (s * sinv).mod(q) = 1 and
    s = (kinv * (h + x * r)).mod(q)
    implies
        (g.pow((h * sinv).mod(q)) * y.pow((r * sinv).mod(q))).mod(p) =
            g.pow(k).mod(p)
}

/// DSA verifier expression equals the signing `r` component under the
/// hypotheses used by DSA correctness.
theorem dsa_verifier_mod_q_eq_r(p: Nat, q: Nat, g: Nat, x: Nat, y: Nat,
                                  h: Nat, k: Nat, kinv: Nat, sinv: Nat) {
    Nat.1 < q and
    g.pow(q).mod(p) = 1 and
    y = g.pow(x).mod(p) and
    k < q and
    (k * kinv).mod(q) = 1 and
    (dsa_sign_s(q, h, x, dsa_sign_r(p, q, g, k), kinv) * sinv).mod(q) = 1
    implies
        (g.pow((h * sinv).mod(q)) *
         y.pow((dsa_sign_r(p, q, g, k) * sinv).mod(q))).mod(p).mod(q) =
            dsa_sign_r(p, q, g, k)
}

/// DSA correctness: a signature produced by `dsa_sign_r` / `dsa_sign_s`
/// verifies under `dsa_verify` for the matching public key.
theorem dsa_correctness(p: Nat, q: Nat, g: Nat, x: Nat, y: Nat,
                        k: Nat, kinv: Nat, h: Nat, sinv: Nat) {
    is_dsa_params(p, q, g) and
    is_dsa_keypair(p, q, g, x, y) and
    0 < k and k < q and
    is_mod_inverse(k, kinv, q) and
    is_mod_inverse(dsa_sign_s(q, h, x, dsa_sign_r(p, q, g, k), kinv), sinv, q) and
    0 < dsa_sign_r(p, q, g, k) and
    0 < dsa_sign_s(q, h, x, dsa_sign_r(p, q, g, k), kinv)
    implies
        dsa_verify(p, q, g, y, h,
                   dsa_sign_r(p, q, g, k),
                   dsa_sign_s(q, h, x, dsa_sign_r(p, q, g, k), kinv),
                   sinv)
}

/// The ECDSA group element extractor: in standard ECDSA this is the
/// `x`-coordinate of an elliptic curve point reduced modulo `n`. Here it is
/// abstracted as an arbitrary function from the group `G` to `Nat`.
typeclass X: XCoord extends Group {
    /// Reduces a group element to a scalar in `[0, n)`, where `n` is the
    /// group order.
    x_coord: X -> Nat
}

/// True if `pub` is the ECDSA public key for private key `d` under generator
/// `gen`. The public key is the `d`-th power of the generator.
define is_ecdsa_keypair[G: XCoord](gen: G, d: Nat, pub: G) -> Bool {
    pub = gen.pow(d)
}

/// Bridge: ECDSA key-pair validity unfolds to the public-key equation.
theorem is_ecdsa_keypair_unfold[G: XCoord](gen: G, d: Nat, pub: G) {
    is_ecdsa_keypair(gen, d, pub) = (pub = gen.pow(d))
}

/// True if `kinv` is the modular inverse of `k` modulo `n`.
define is_mod_inv(k: Nat, kinv: Nat, n: Nat) -> Bool {
    (k * kinv).mod(n) = 1
}

/// Bridge: modular-inverse validity is the defining product congruence.
theorem is_mod_inv_unfold(k: Nat, kinv: Nat, n: Nat) {
    is_mod_inv(k, kinv, n) = ((k * kinv).mod(n) = 1)
}

/// The first ECDSA signature component:
/// `r = x_coord(k * gen) mod n`.
define ecdsa_sign_r[G: XCoord](n: Nat, gen: G, k: Nat) -> Nat {
    gen.pow(k).x_coord.mod(n)
}

/// The second ECDSA signature component:
/// `s = k^{-1} (h + r * d) mod n`.
define ecdsa_sign_s(n: Nat, h: Nat, d: Nat, r: Nat, kinv: Nat) -> Nat {
    (kinv * (h + r * d)).mod(n)
}

/// The ECDSA `r` component is reduced modulo `n`.
theorem ecdsa_sign_r_lt[G: XCoord](n: Nat, gen: G, k: Nat) {
    n != 0 implies ecdsa_sign_r(n, gen, k) < n
}

/// The ECDSA `s` component is reduced modulo `n`.
theorem ecdsa_sign_s_lt(n: Nat, h: Nat, d: Nat, r: Nat, kinv: Nat) {
    n != 0 implies ecdsa_sign_s(n, h, d, r, kinv) < n
}

/// ECDSA signature verification.
/// Accept `(r, s)` for hash `h` under public key `pub`, generator `gen`,
/// and group order `n` when, for `w = s^{-1} mod n`, the point
/// `gen.pow(h * w) * pub.pow(r * w)` has `x_coord` congruent to `r` mod `n`.
define ecdsa_verify[G: XCoord](n: Nat, gen: G, pub: G, h: Nat,
                               r: Nat, s: Nat, sinv: Nat) -> Bool {
    0 < r and r < n and 0 < s and s < n and
    is_mod_inv(s, sinv, n) and
    (gen.pow((h * sinv).mod(n)) * pub.pow((r * sinv).mod(n))).x_coord.mod(n) = r
}
