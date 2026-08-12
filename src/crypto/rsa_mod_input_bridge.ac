from crypto.rsa import Nat, rsa_encrypt, rsa_decrypt
from number_theory import congr_mod_pow, mod_congr_mod_self
numerals Nat

/// RSA encryption is unchanged by reducing the plaintext modulo the modulus first.
theorem rsa_encrypt_mod_input(m: Nat, e: Nat, n: Nat) {
    rsa_encrypt(m.mod(n), e, n) = rsa_encrypt(m, e, n)
} by {
    mod_congr_mod_self(m, n)
    m.mod(n).congr_mod(m, n)
    congr_mod_pow(m.mod(n), m, n, e)
    m.mod(n).pow(e).congr_mod(m.pow(e), n)
    rsa_encrypt(m.mod(n), e, n) = m.mod(n).pow(e).mod(n)
    rsa_encrypt(m, e, n) = m.pow(e).mod(n)
    m.mod(n).pow(e).mod(n) = m.pow(e).mod(n)
}

/// RSA decryption is unchanged by reducing the ciphertext modulo the modulus first.
theorem rsa_decrypt_mod_input(c: Nat, d: Nat, n: Nat) {
    rsa_decrypt(c.mod(n), d, n) = rsa_decrypt(c, d, n)
} by {
    mod_congr_mod_self(c, n)
    c.mod(n).congr_mod(c, n)
    congr_mod_pow(c.mod(n), c, n, d)
    c.mod(n).pow(d).congr_mod(c.pow(d), n)
    rsa_decrypt(c.mod(n), d, n) = c.mod(n).pow(d).mod(n)
    rsa_decrypt(c, d, n) = c.pow(d).mod(n)
    c.mod(n).pow(d).mod(n) = c.pow(d).mod(n)
}

/// RSA encryption depends only on the plaintext residue modulo the modulus.
theorem rsa_encrypt_respects_congr_mod(m1: Nat, m2: Nat, e: Nat, n: Nat) {
    m1.congr_mod(m2, n) implies rsa_encrypt(m1, e, n) = rsa_encrypt(m2, e, n)
} by {
    if m1.congr_mod(m2, n) {
        congr_mod_pow(m1, m2, n, e)
        m1.pow(e).congr_mod(m2.pow(e), n)
        rsa_encrypt(m1, e, n) = m1.pow(e).mod(n)
        rsa_encrypt(m2, e, n) = m2.pow(e).mod(n)
        m1.pow(e).mod(n) = m2.pow(e).mod(n)
    }
}

/// RSA decryption depends only on the ciphertext residue modulo the modulus.
theorem rsa_decrypt_respects_congr_mod(c1: Nat, c2: Nat, d: Nat, n: Nat) {
    c1.congr_mod(c2, n) implies rsa_decrypt(c1, d, n) = rsa_decrypt(c2, d, n)
} by {
    if c1.congr_mod(c2, n) {
        congr_mod_pow(c1, c2, n, d)
        c1.pow(d).congr_mod(c2.pow(d), n)
        rsa_decrypt(c1, d, n) = c1.pow(d).mod(n)
        rsa_decrypt(c2, d, n) = c2.pow(d).mod(n)
        c1.pow(d).mod(n) = c2.pow(d).mod(n)
    }
}
