from nat import Nat
from algebra.group import Group
from number_theory import mod_lt
numerals Nat

/// The ECDSA group element extractor: in standard ECDSA this is the
/// `x`-coordinate of an elliptic curve point reduced modulo `n`. Here it is
/// abstracted as an arbitrary function from the group `G` to `Nat`.
typeclass X: XCoord extends Group {
    /// Reduces a group element to a scalar in `[0, n)`, where `n` is the
    /// group order.
    x_coord: X -> Nat
}

/// True if `pub` is the ECDSA public key for private key `d` under generator
/// `gen`. The public key is the `d`-th power of the generator.
define is_ecdsa_keypair[G: XCoord](gen: G, d: Nat, pub: G) -> Bool {
    pub = gen.pow(d)
}

/// Bridge: ECDSA key-pair validity unfolds to the public-key equation.
theorem is_ecdsa_keypair_unfold[G: XCoord](gen: G, d: Nat, pub: G) {
    is_ecdsa_keypair(gen, d, pub) = (pub = gen.pow(d))
}

/// True if `kinv` is the modular inverse of `k` modulo `n`.
define is_mod_inv(k: Nat, kinv: Nat, n: Nat) -> Bool {
    (k * kinv).mod(n) = 1
}

/// Bridge: modular-inverse validity is the defining product congruence.
theorem is_mod_inv_unfold(k: Nat, kinv: Nat, n: Nat) {
    is_mod_inv(k, kinv, n) = ((k * kinv).mod(n) = 1)
}

/// The first ECDSA signature component:
/// `r = x_coord(k * gen) mod n`.
define ecdsa_sign_r[G: XCoord](n: Nat, gen: G, k: Nat) -> Nat {
    gen.pow(k).x_coord.mod(n)
}

/// The second ECDSA signature component:
/// `s = k^{-1} (h + r * d) mod n`.
define ecdsa_sign_s(n: Nat, h: Nat, d: Nat, r: Nat, kinv: Nat) -> Nat {
    (kinv * (h + r * d)).mod(n)
}

/// The ECDSA `r` component is reduced modulo `n`.
theorem ecdsa_sign_r_lt[G: XCoord](n: Nat, gen: G, k: Nat) {
    n != 0 implies ecdsa_sign_r(n, gen, k) < n
} by {
    if n != 0 {
        ecdsa_sign_r(n, gen, k) = gen.pow(k).x_coord.mod(n)
        mod_lt(gen.pow(k).x_coord, n)
    }
}

/// The ECDSA `s` component is reduced modulo `n`.
theorem ecdsa_sign_s_lt(n: Nat, h: Nat, d: Nat, r: Nat, kinv: Nat) {
    n != 0 implies ecdsa_sign_s(n, h, d, r, kinv) < n
} by {
    if n != 0 {
        ecdsa_sign_s(n, h, d, r, kinv) = (kinv * (h + r * d)).mod(n)
        mod_lt(kinv * (h + r * d), n)
    }
}

/// ECDSA signature verification.
/// Accept `(r, s)` for hash `h` under public key `pub`, generator `gen`,
/// and group order `n` when, for `w = s^{-1} mod n`, the point
/// `gen.pow(h * w) * pub.pow(r * w)` has `x_coord` congruent to `r` mod `n`.
define ecdsa_verify[G: XCoord](n: Nat, gen: G, pub: G, h: Nat,
                               r: Nat, s: Nat, sinv: Nat) -> Bool {
    0 < r and r < n and 0 < s and s < n and
    is_mod_inv(s, sinv, n) and
    (gen.pow((h * sinv).mod(n)) * pub.pow((r * sinv).mod(n))).x_coord.mod(n) = r
}

// TODO: prove ECDSA correctness. Requires:
//   - a finite cyclic group of prime order `n` (mathlib's CyclicGroup);
//   - the identity `gen.pow(a).pow(b) = gen.pow(a * b)` and reductions
//     `gen.pow(k mod n) = gen.pow(k)` when `gen` has order `n`;
//   - modular inverse arithmetic;
//   - an instance of `XCoord` for an actual elliptic curve group.
// None of these are currently in acornlib, and the elliptic-curve instance
// in particular is a major piece of work.
//
// /// ECDSA correctness: a signature produced by `ecdsa_sign_r` /
// /// `ecdsa_sign_s` verifies under `ecdsa_verify` for the matching public key.
// theorem ecdsa_correctness[G: XCoord](n: Nat, gen: G, d: Nat, pub: G,
//                                      k: Nat, kinv: Nat, h: Nat, sinv: Nat) {
//     1 < n and
//     is_ecdsa_keypair(gen, d, pub) and
//     0 < d and d < n and
//     0 < k and k < n and
//     is_mod_inv(k, kinv, n) and
//     is_mod_inv(ecdsa_sign_s(n, h, d, ecdsa_sign_r(n, gen, k), kinv), sinv, n) and
//     0 < ecdsa_sign_r(n, gen, k) and
//     0 < ecdsa_sign_s(n, h, d, ecdsa_sign_r(n, gen, k), kinv)
//     implies
//         ecdsa_verify(n, gen, pub, h,
//                      ecdsa_sign_r(n, gen, k),
//                      ecdsa_sign_s(n, h, d, ecdsa_sign_r(n, gen, k), kinv),
//                      sinv)
// }
