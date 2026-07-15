from nat import Nat
from nat import add_mod, small_mod
from number_theory import congr_mod_pow, congr_mod_mul, congr_mod_refl,
    mod_congr_mod_self, congr_mod_symm, congr_mod_trans, congr_mod_add, mod_lt
from number_theory import prime_imp_no_proper_divisor
numerals Nat

/// If `g` has order dividing `q` modulo `p` (i.e. `g^q ≡ 1 (mod p)`),
/// then raising `g` to `k * q + r` is congruent modulo `p` to raising
/// `g` to `r`.
theorem dsa_pow_order_reduce(p: Nat, q: Nat, g: Nat, k: Nat, r: Nat) {
    g.pow(q).mod(p) = 1 implies g.pow(k * q + r).mod(p) = g.pow(r).mod(p)
} by {
    if g.pow(q).mod(p) = 1 {
        g.pow(q).congr_mod(Nat.1, p)
        congr_mod_pow(g.pow(q), Nat.1, p, k)
        congr_mod_refl(g.pow(r), p)
        congr_mod_mul(g.pow(q).pow(k), g.pow(r), Nat.1, g.pow(r), p)
        (g.pow(q).pow(k) * g.pow(r)).congr_mod(Nat.1 * g.pow(r), p)
        (g.pow(q).pow(k) * g.pow(r)).mod(p) = g.pow(r).mod(p)
        g.pow(k * q + r).mod(p) = g.pow(r).mod(p)
    }
}

/// If `g` has order dividing `q` modulo `p`, then `g.pow(a).mod(p)` only
/// depends on `a` modulo `q`.
theorem dsa_pow_mod_q(p: Nat, q: Nat, g: Nat, a: Nat) {
    g.pow(q).mod(p) = 1 implies g.pow(a).mod(p) = g.pow(a.mod(q)).mod(p)
} by {
    if g.pow(q).mod(p) = 1 {
        add_mod(a, q)
        let k: Nat satisfy { k * q + a.mod(q) = a }
        dsa_pow_order_reduce(p, q, g, k, a.mod(q))
        g.pow(a).mod(p) = g.pow(a.mod(q)).mod(p)
    }
}

/// If `y = g^x mod p`, then `y^b` is congruent to `g^(x*b)` modulo `p`.
theorem dsa_y_pow_congr(p: Nat, g: Nat, x: Nat, y: Nat, b: Nat) {
    y = g.pow(x).mod(p) implies y.pow(b).congr_mod(g.pow(x * b), p)
} by {
    if y = g.pow(x).mod(p) {
        mod_congr_mod_self(g.pow(x), p)
        congr_mod_pow(y, g.pow(x), p, b)
        y.pow(b).congr_mod(g.pow(x).pow(b), p)
        y.pow(b).congr_mod(g.pow(x * b), p)
    }
}

/// If `y = g^x mod p`, then `(g^a * y^b) mod p = g^(a + x*b) mod p`.
theorem dsa_combine_pow(p: Nat, g: Nat, x: Nat, y: Nat, a: Nat, b: Nat) {
    y = g.pow(x).mod(p) implies
        (g.pow(a) * y.pow(b)).mod(p) = g.pow(a + x * b).mod(p)
} by {
    if y = g.pow(x).mod(p) {
        dsa_y_pow_congr(p, g, x, y, b)
        congr_mod_refl(g.pow(a), p)
        congr_mod_mul(g.pow(a), y.pow(b), g.pow(a), g.pow(x * b), p)
        (g.pow(a) * y.pow(b)).mod(p) = g.pow(a + x * b).mod(p)
    }
}

/// Mod-q normalization of the verifier's combined exponent: pulling the
/// inner `.mod(q)` out of each summand leaves the result unchanged
/// modulo `q`, and the outer expression equals `(h + x*r)*sinv mod q`.
theorem dsa_exp_mod_q_factor(q: Nat, h: Nat, x: Nat, r: Nat, sinv: Nat) {
    ((h * sinv).mod(q) + x * (r * sinv).mod(q)).mod(q) =
        ((h + x * r) * sinv).mod(q)
} by {
    mod_congr_mod_self(h * sinv, q)
    mod_congr_mod_self(r * sinv, q)
    congr_mod_refl(x, q)
    congr_mod_mul(x, (r * sinv).mod(q), x, r * sinv, q)
    congr_mod_add((h * sinv).mod(q), x * (r * sinv).mod(q),
                  h * sinv, x * (r * sinv), q)
    h * sinv + x * (r * sinv) = (h + x * r) * sinv
}

/// Cancellation step: `s ≡ kinv*(h + x*r) (mod q)` together with
/// `(k*kinv) ≡ 1 (mod q)` forces `k*s ≡ h + x*r (mod q)`.
theorem dsa_k_times_s_congr(q: Nat, h: Nat, x: Nat, r: Nat,
                             k: Nat, kinv: Nat, s: Nat) {
    s.congr_mod(kinv * (h + x * r), q) and (k * kinv).congr_mod(Nat.1, q)
    implies (k * s).congr_mod(h + x * r, q)
} by {
    if s.congr_mod(kinv * (h + x * r), q) and (k * kinv).congr_mod(Nat.1, q) {
        congr_mod_refl(k, q)
        congr_mod_mul(k, s, k, kinv * (h + x * r), q)
        congr_mod_refl(h + x * r, q)
        congr_mod_mul(k * kinv, h + x * r, Nat.1, h + x * r, q)
        ((k * kinv) * (h + x * r)).congr_mod(h + x * r, q)
        congr_mod_trans(k * s, (k * kinv) * (h + x * r), h + x * r, q)
        (k * s).congr_mod(h + x * r, q)
    }
}

/// Multiplying `(k*s).congr_mod(h + x*r, q)` by `sinv` and applying
/// `(s*sinv).congr_mod(1, q)` yields `((h + x*r)*sinv).congr_mod(k, q)`.
theorem dsa_finish_congr(q: Nat, h: Nat, x: Nat, r: Nat,
                          k: Nat, s: Nat, sinv: Nat) {
    (k * s).congr_mod(h + x * r, q) and (s * sinv).congr_mod(Nat.1, q)
    implies ((h + x * r) * sinv).congr_mod(k, q)
} by {
    if (k * s).congr_mod(h + x * r, q) and (s * sinv).congr_mod(Nat.1, q) {
        congr_mod_symm(k * s, h + x * r, q)
        congr_mod_refl(sinv, q)
        congr_mod_mul(h + x * r, sinv, k * s, sinv, q)
        ((h + x * r) * sinv).congr_mod(k * s * sinv, q)
        congr_mod_refl(k, q)
        congr_mod_mul(k, s * sinv, k, Nat.1, q)
        (k * (s * sinv)).congr_mod(k, q)
        congr_mod_trans((h + x * r) * sinv, k * (s * sinv), k, q)
        ((h + x * r) * sinv).congr_mod(k, q)
    }
}

/// Bridge: a congruence `lhs ≡ k (mod q)` together with `k < q` upgrades
/// to the equality `lhs.mod(q) = k`.
theorem dsa_congr_to_eq(q: Nat, k: Nat, lhs: Nat) {
    k < q and lhs.congr_mod(k, q) implies lhs.mod(q) = k
} by {
    if k < q and lhs.congr_mod(k, q) {
        small_mod(k, q)
        lhs.mod(q) = k
    }
}

/// Lifted form of `dsa_k_times_s_congr`: derives `(k*s) ≡ h + x*r (mod q)`
/// directly from the integer mod hypotheses used in DSA.
theorem dsa_k_times_s_from_mod(q: Nat, h: Nat, x: Nat, r: Nat,
                                k: Nat, kinv: Nat, s: Nat) {
    1 < q and (k * kinv).mod(q) = 1 and
    s = (kinv * (h + x * r)).mod(q)
    implies (k * s).congr_mod(h + x * r, q)
} by {
    if 1 < q and (k * kinv).mod(q) = 1 and
       s = (kinv * (h + x * r)).mod(q) {
        small_mod(Nat.1, q)
        mod_congr_mod_self(kinv * (h + x * r), q)
        (k * kinv).congr_mod(Nat.1, q)
        dsa_k_times_s_congr(q, h, x, r, k, kinv, s)
        (k * s).congr_mod(h + x * r, q)
    }
}

/// DSA domain parameters: a prime `p`, a prime `q` dividing `p - 1`,
/// and a generator `g` of the unique order-`q` subgroup of `(Z/p)*`.
define is_dsa_params(p: Nat, q: Nat, g: Nat) -> Bool {
    p.is_prime and q.is_prime and q.divides(p - 1) and
    1 < g and g < p and g.pow(q).mod(p) = 1
}

/// Bridge: DSA parameter validity unfolds to its arithmetic and order clauses.
theorem is_dsa_params_unfold(p: Nat, q: Nat, g: Nat) {
    is_dsa_params(p, q, g) = (p.is_prime and q.is_prime and
        q.divides(p - 1) and 1 < g and g < p and g.pow(q).mod(p) = 1)
}

/// True if `y` is the DSA public key corresponding to private key `x`
/// under parameters `(p, q, g)`.
define is_dsa_keypair(p: Nat, q: Nat, g: Nat, x: Nat, y: Nat) -> Bool {
    0 < x and x < q and y = g.pow(x).mod(p)
}

/// Bridge: DSA key-pair validity unfolds to the private-key bounds and
/// public-key exponent equation.
theorem is_dsa_keypair_unfold(p: Nat, q: Nat, g: Nat, x: Nat, y: Nat) {
    is_dsa_keypair(p, q, g, x, y) = (0 < x and x < q and y = g.pow(x).mod(p))
}

/// True if `kinv` is the modular inverse of `k` modulo `q`.
define is_mod_inverse(k: Nat, kinv: Nat, q: Nat) -> Bool {
    (k * kinv).mod(q) = 1
}

/// Bridge: DSA modular-inverse validity is the defining product congruence.
theorem is_mod_inverse_unfold(k: Nat, kinv: Nat, q: Nat) {
    is_mod_inverse(k, kinv, q) = ((k * kinv).mod(q) = 1)
}

/// The first DSA signature component: `r = (g^k mod p) mod q`.
define dsa_sign_r(p: Nat, q: Nat, g: Nat, k: Nat) -> Nat {
    g.pow(k).mod(p).mod(q)
}

/// The second DSA signature component:
/// `s = k^{-1} (h + x * r) mod q`, given a precomputed inverse `kinv` of `k`.
define dsa_sign_s(q: Nat, h: Nat, x: Nat, r: Nat, kinv: Nat) -> Nat {
    (kinv * (h + x * r)).mod(q)
}

/// The DSA `r` component is reduced modulo `q`.
theorem dsa_sign_r_lt(p: Nat, q: Nat, g: Nat, k: Nat) {
    q != 0 implies dsa_sign_r(p, q, g, k) < q
} by {
    if q != 0 {
        dsa_sign_r(p, q, g, k) = g.pow(k).mod(p).mod(q)
        mod_lt(g.pow(k).mod(p), q)
    }
}

/// The DSA `s` component is reduced modulo `q`.
theorem dsa_sign_s_lt(q: Nat, h: Nat, x: Nat, r: Nat, kinv: Nat) {
    q != 0 implies dsa_sign_s(q, h, x, r, kinv) < q
} by {
    if q != 0 {
        dsa_sign_s(q, h, x, r, kinv) = (kinv * (h + x * r)).mod(q)
        mod_lt(kinv * (h + x * r), q)
    }
}

/// DSA signature verification:
/// accept `(r, s)` for hash `h` under public key `y` and parameters `(p, q, g)`
/// when `r = (g^{h * w} * y^{r * w} mod p) mod q` for `w = s^{-1} mod q`.
define dsa_verify(p: Nat, q: Nat, g: Nat, y: Nat, h: Nat,
                  r: Nat, s: Nat, sinv: Nat) -> Bool {
    0 < r and r < q and 0 < s and s < q and
    is_mod_inverse(s, sinv, q) and
    (g.pow((h * sinv).mod(q)) * y.pow((r * sinv).mod(q))).mod(p).mod(q) = r
}

/// Helper for `dsa_verify_identity`: the "exponent collapses to `k`" step.
/// Given the modular-inverse relations and `s = kinv*(h+x*r) mod q`, the
/// reduced exponent `((h*sinv).mod(q) + x*(r*sinv).mod(q)).mod(q)` equals `k`.
theorem dsa_verify_exponent_eq(q: Nat, h: Nat, x: Nat, r: Nat,
                                k: Nat, kinv: Nat, s: Nat, sinv: Nat) {
    1 < q and k < q and
    (k * kinv).mod(q) = 1 and
    (s * sinv).mod(q) = 1 and
    s = (kinv * (h + x * r)).mod(q)
    implies
        ((h * sinv).mod(q) + x * (r * sinv).mod(q)).mod(q) = k
} by {
    if 1 < q and k < q and
       (k * kinv).mod(q) = 1 and
       (s * sinv).mod(q) = 1 and
       s = (kinv * (h + x * r)).mod(q) {
        dsa_exp_mod_q_factor(q, h, x, r, sinv)
        dsa_k_times_s_from_mod(q, h, x, r, k, kinv, s)
        small_mod(Nat.1, q)
        (s * sinv).congr_mod(Nat.1, q)
        dsa_finish_congr(q, h, x, r, k, s, sinv)
        dsa_congr_to_eq(q, k, (h + x * r) * sinv)
        ((h + x * r) * sinv).mod(q) = k
        ((h * sinv).mod(q) + x * (r * sinv).mod(q)).mod(q) = k
    }
}

/// Bundled algebraic step used by `dsa_verify_identity`. If `g^q ≡ 1 (mod p)`,
/// `y = g^x mod p`, and the exponent `a + x*b` reduces modulo `q` to `k`, then
/// `(g^a * y^b) mod p = g^k mod p`.
theorem dsa_pow_combine(p: Nat, q: Nat, g: Nat, x: Nat, y: Nat,
                         a: Nat, b: Nat, k: Nat) {
    y = g.pow(x).mod(p) and
    g.pow(q).mod(p) = 1 and
    (a + x * b).mod(q) = k
    implies
        (g.pow(a) * y.pow(b)).mod(p) = g.pow(k).mod(p)
} by {
    if y = g.pow(x).mod(p) and
       g.pow(q).mod(p) = 1 and
       (a + x * b).mod(q) = k {
        dsa_combine_pow(p, g, x, y, a, b)
        dsa_pow_mod_q(p, q, g, a + x * b)
        (g.pow(a) * y.pow(b)).mod(p) = g.pow(k).mod(p)
    }
}

/// The DSA verification identity. Under the order-`q` subgroup hypothesis
/// `g^q ≡ 1 (mod p)`, the public-key relation `y = g^x mod p`, and the
/// modular-inverse hypotheses linking `k`, `kinv`, `s`, `sinv` with
/// `s = kinv*(h + x*r) mod q`, the verifier expression
/// `(g^{h*sinv mod q} * y^{r*sinv mod q}) mod p` collapses to `g^k mod p`.
theorem dsa_verify_identity(p: Nat, q: Nat, g: Nat, x: Nat, y: Nat,
                             h: Nat, r: Nat, k: Nat, kinv: Nat,
                             s: Nat, sinv: Nat) {
    1 < q and
    g.pow(q).mod(p) = 1 and
    y = g.pow(x).mod(p) and
    k < q and
    (k * kinv).mod(q) = 1 and
    (s * sinv).mod(q) = 1 and
    s = (kinv * (h + x * r)).mod(q)
    implies
        (g.pow((h * sinv).mod(q)) * y.pow((r * sinv).mod(q))).mod(p) =
            g.pow(k).mod(p)
} by {
    if 1 < q and
       g.pow(q).mod(p) = 1 and
       y = g.pow(x).mod(p) and
       k < q and
       (k * kinv).mod(q) = 1 and
       (s * sinv).mod(q) = 1 and
       s = (kinv * (h + x * r)).mod(q) {
        dsa_verify_exponent_eq(q, h, x, r, k, kinv, s, sinv)
        dsa_pow_combine(p, q, g, x, y,
                         (h * sinv).mod(q), (r * sinv).mod(q), k)
        (g.pow((h * sinv).mod(q)) * y.pow((r * sinv).mod(q))).mod(p) =
            g.pow(k).mod(p)
    }
}

/// Verifier expression equals `r`. Bridges `dsa_verify_identity` with the
/// `dsa_sign_r` definition by chaining `(...).mod(p) = g.pow(k).mod(p)` with
/// `r = g.pow(k).mod(p).mod(q)`.
theorem dsa_verifier_mod_q_eq_r(p: Nat, q: Nat, g: Nat, x: Nat, y: Nat,
                                 h: Nat, k: Nat, kinv: Nat, sinv: Nat) {
    Nat.1 < q and
    g.pow(q).mod(p) = 1 and
    y = g.pow(x).mod(p) and
    k < q and
    (k * kinv).mod(q) = 1 and
    (dsa_sign_s(q, h, x, dsa_sign_r(p, q, g, k), kinv) * sinv).mod(q) = 1
    implies
        (g.pow((h * sinv).mod(q)) *
         y.pow((dsa_sign_r(p, q, g, k) * sinv).mod(q))).mod(p).mod(q) =
            dsa_sign_r(p, q, g, k)
} by {
    if Nat.1 < q and
       g.pow(q).mod(p) = 1 and
       y = g.pow(x).mod(p) and
       k < q and
       (k * kinv).mod(q) = 1 and
       (dsa_sign_s(q, h, x, dsa_sign_r(p, q, g, k), kinv) * sinv).mod(q) = 1 {
        let r: Nat = dsa_sign_r(p, q, g, k)
        let s: Nat = dsa_sign_s(q, h, x, r, kinv)
        (s * sinv).mod(q) = 1
        dsa_verify_identity(p, q, g, x, y, h, r, k, kinv, s, sinv)
        (g.pow((h * sinv).mod(q)) * y.pow((r * sinv).mod(q))).mod(p).mod(q) =
            g.pow(k).mod(p).mod(q)
        (g.pow((h * sinv).mod(q)) *
         y.pow((dsa_sign_r(p, q, g, k) * sinv).mod(q))).mod(p).mod(q) =
            dsa_sign_r(p, q, g, k)
    }
}

/// DSA correctness: a signature produced by `dsa_sign_r` / `dsa_sign_s`
/// verifies under `dsa_verify` for the matching public key.
theorem dsa_correctness(p: Nat, q: Nat, g: Nat, x: Nat, y: Nat,
                        k: Nat, kinv: Nat, h: Nat, sinv: Nat) {
    is_dsa_params(p, q, g) and
    is_dsa_keypair(p, q, g, x, y) and
    0 < k and k < q and
    is_mod_inverse(k, kinv, q) and
    is_mod_inverse(dsa_sign_s(q, h, x, dsa_sign_r(p, q, g, k), kinv), sinv, q) and
    0 < dsa_sign_r(p, q, g, k) and
    0 < dsa_sign_s(q, h, x, dsa_sign_r(p, q, g, k), kinv)
    implies
        dsa_verify(p, q, g, y, h,
                   dsa_sign_r(p, q, g, k),
                   dsa_sign_s(q, h, x, dsa_sign_r(p, q, g, k), kinv),
                   sinv)
} by {
    if is_dsa_params(p, q, g) and
       is_dsa_keypair(p, q, g, x, y) and
       0 < k and k < q and
       is_mod_inverse(k, kinv, q) and
       is_mod_inverse(dsa_sign_s(q, h, x, dsa_sign_r(p, q, g, k), kinv),
                      sinv, q) and
       0 < dsa_sign_r(p, q, g, k) and
       0 < dsa_sign_s(q, h, x, dsa_sign_r(p, q, g, k), kinv) {
        is_dsa_params(p, q, g) = (p.is_prime and q.is_prime and
            q.divides(p - 1) and Nat.1 < g and g < p and g.pow(q).mod(p) = 1)
        prime_imp_no_proper_divisor(q)
        Nat.1 < q
        q != Nat.0
        is_dsa_keypair(p, q, g, x, y) = (Nat.0 < x and x < q and
            y = g.pow(x).mod(p))
        is_mod_inverse(k, kinv, q) = ((k * kinv).mod(q) = 1)
        is_mod_inverse(dsa_sign_s(q, h, x, dsa_sign_r(p, q, g, k), kinv),
                       sinv, q) =
            ((dsa_sign_s(q, h, x, dsa_sign_r(p, q, g, k), kinv) * sinv).mod(q)
             = 1)
        dsa_verifier_mod_q_eq_r(p, q, g, x, y, h, k, kinv, sinv)
        mod_lt(g.pow(k).mod(p), q)
        mod_lt(kinv * (h + x * dsa_sign_r(p, q, g, k)), q)
        dsa_sign_s(q, h, x, dsa_sign_r(p, q, g, k), kinv) =
            (kinv * (h + x * dsa_sign_r(p, q, g, k))).mod(q)
        dsa_sign_s(q, h, x, dsa_sign_r(p, q, g, k), kinv) < q
        dsa_verify(p, q, g, y, h,
                   dsa_sign_r(p, q, g, k),
                   dsa_sign_s(q, h, x, dsa_sign_r(p, q, g, k), kinv),
                   sinv) = (
            Nat.0 < dsa_sign_r(p, q, g, k) and
            dsa_sign_r(p, q, g, k) < q and
            Nat.0 < dsa_sign_s(q, h, x, dsa_sign_r(p, q, g, k), kinv) and
            dsa_sign_s(q, h, x, dsa_sign_r(p, q, g, k), kinv) < q and
            is_mod_inverse(
                dsa_sign_s(q, h, x, dsa_sign_r(p, q, g, k), kinv), sinv, q) and
            (g.pow((h * sinv).mod(q)) *
             y.pow((dsa_sign_r(p, q, g, k) * sinv).mod(q))).mod(p).mod(q) =
                dsa_sign_r(p, q, g, k))
        dsa_verify(p, q, g, y, h,
                   dsa_sign_r(p, q, g, k),
                   dsa_sign_s(q, h, x, dsa_sign_r(p, q, g, k), kinv),
                   sinv)
    }
}
