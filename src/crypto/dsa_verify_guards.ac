/// Guard predicates and API bridges for DSA signature verification.

from crypto.dsa import Nat, is_mod_inverse, dsa_verify
from nat import not_lt_zero

numerals Nat

/// The public range check on the DSA `r` scalar.
define dsa_r_in_range(q: Nat, r: Nat) -> Bool {
    0 < r and r < q
}

/// The public range check on the DSA `s` scalar.
define dsa_s_in_range(q: Nat, s: Nat) -> Bool {
    0 < s and s < q
}

/// The public range checks on the two DSA signature scalars.
define dsa_signature_components_in_range(q: Nat, r: Nat, s: Nat) -> Bool {
    dsa_r_in_range(q, r) and dsa_s_in_range(q, s)
}

/// The DSA verifier's final modular equation.
define dsa_verify_core_equation(
    p: Nat,
    q: Nat,
    g: Nat,
    y: Nat,
    h: Nat,
    r: Nat,
    sinv: Nat
) -> Bool {
    (g.pow((h * sinv).mod(q)) * y.pow((r * sinv).mod(q))).mod(p).mod(q) = r
}

/// The DSA `r` range guard gives positivity of `r`.
theorem dsa_r_in_range_pos(q: Nat, r: Nat) {
    dsa_r_in_range(q, r) implies 0 < r
} by {
    if dsa_r_in_range(q, r) {
        dsa_r_in_range(q, r) = (0 < r and r < q)
        0 < r
    }
}

/// The DSA `r` range guard gives the upper bound on `r`.
theorem dsa_r_in_range_lt(q: Nat, r: Nat) {
    dsa_r_in_range(q, r) implies r < q
} by {
    if dsa_r_in_range(q, r) {
        dsa_r_in_range(q, r) = (0 < r and r < q)
        r < q
    }
}

/// The DSA `s` range guard gives positivity of `s`.
theorem dsa_s_in_range_pos(q: Nat, s: Nat) {
    dsa_s_in_range(q, s) implies 0 < s
} by {
    if dsa_s_in_range(q, s) {
        dsa_s_in_range(q, s) = (0 < s and s < q)
        0 < s
    }
}

/// The DSA `s` range guard gives the upper bound on `s`.
theorem dsa_s_in_range_lt(q: Nat, s: Nat) {
    dsa_s_in_range(q, s) implies s < q
} by {
    if dsa_s_in_range(q, s) {
        dsa_s_in_range(q, s) = (0 < s and s < q)
        s < q
    }
}

/// The combined range guard gives the `r` range guard.
theorem dsa_signature_components_in_range_r(q: Nat, r: Nat, s: Nat) {
    dsa_signature_components_in_range(q, r, s) implies dsa_r_in_range(q, r)
} by {
    if dsa_signature_components_in_range(q, r, s) {
        dsa_signature_components_in_range(q, r, s) =
            (dsa_r_in_range(q, r) and dsa_s_in_range(q, s))
        dsa_r_in_range(q, r)
    }
}

/// The combined range guard gives the `s` range guard.
theorem dsa_signature_components_in_range_s(q: Nat, r: Nat, s: Nat) {
    dsa_signature_components_in_range(q, r, s) implies dsa_s_in_range(q, s)
} by {
    if dsa_signature_components_in_range(q, r, s) {
        dsa_signature_components_in_range(q, r, s) =
            (dsa_r_in_range(q, r) and dsa_s_in_range(q, s))
        dsa_s_in_range(q, s)
    }
}

/// The DSA signature-component range guard gives positivity of `r`.
theorem dsa_signature_components_in_range_r_pos(q: Nat, r: Nat, s: Nat) {
    dsa_signature_components_in_range(q, r, s) implies 0 < r
} by {
    if dsa_signature_components_in_range(q, r, s) {
        dsa_signature_components_in_range_r(q, r, s)
        dsa_r_in_range(q, r)
        dsa_r_in_range_pos(q, r)
        0 < r
    }
}

/// The DSA signature-component range guard gives the upper bound on `r`.
theorem dsa_signature_components_in_range_r_lt(q: Nat, r: Nat, s: Nat) {
    dsa_signature_components_in_range(q, r, s) implies r < q
} by {
    if dsa_signature_components_in_range(q, r, s) {
        dsa_signature_components_in_range_r(q, r, s)
        dsa_r_in_range(q, r)
        dsa_r_in_range_lt(q, r)
        r < q
    }
}

/// The DSA signature-component range guard gives positivity of `s`.
theorem dsa_signature_components_in_range_s_pos(q: Nat, r: Nat, s: Nat) {
    dsa_signature_components_in_range(q, r, s) implies 0 < s
} by {
    if dsa_signature_components_in_range(q, r, s) {
        dsa_signature_components_in_range_s(q, r, s)
        dsa_s_in_range(q, s)
        dsa_s_in_range_pos(q, s)
        0 < s
    }
}

/// The DSA signature-component range guard gives the upper bound on `s`.
theorem dsa_signature_components_in_range_s_lt(q: Nat, r: Nat, s: Nat) {
    dsa_signature_components_in_range(q, r, s) implies s < q
} by {
    if dsa_signature_components_in_range(q, r, s) {
        dsa_signature_components_in_range_s(q, r, s)
        dsa_s_in_range(q, s)
        dsa_s_in_range_lt(q, s)
        s < q
    }
}

/// Verification implies the public signature range guard and the inverse guard.
theorem dsa_verify_guards_of_verify(
    p: Nat,
    q: Nat,
    g: Nat,
    y: Nat,
    h: Nat,
    r: Nat,
    s: Nat,
    sinv: Nat
) {
    dsa_verify(p, q, g, y, h, r, s, sinv) implies
        dsa_signature_components_in_range(q, r, s) and is_mod_inverse(s, sinv, q)
} by {
    if dsa_verify(p, q, g, y, h, r, s, sinv) {
        dsa_verify(p, q, g, y, h, r, s, sinv) =
            (0 < r and r < q and 0 < s and s < q and
             is_mod_inverse(s, sinv, q) and
             (g.pow((h * sinv).mod(q)) * y.pow((r * sinv).mod(q))).mod(p).mod(q) = r)
        dsa_r_in_range(q, r)
        dsa_s_in_range(q, s)
        dsa_signature_components_in_range(q, r, s)
        is_mod_inverse(s, sinv, q)
        dsa_signature_components_in_range(q, r, s) and is_mod_inverse(s, sinv, q)
    }
}

/// Verification implies the public signature range guard.
theorem dsa_verify_signature_components_in_range_of_verify(
    p: Nat,
    q: Nat,
    g: Nat,
    y: Nat,
    h: Nat,
    r: Nat,
    s: Nat,
    sinv: Nat
) {
    dsa_verify(p, q, g, y, h, r, s, sinv) implies dsa_signature_components_in_range(q, r, s)
} by {
    if dsa_verify(p, q, g, y, h, r, s, sinv) {
        dsa_verify_guards_of_verify(p, q, g, y, h, r, s, sinv)
        dsa_signature_components_in_range(q, r, s)
    }
}

/// Verification implies the modular-inverse guard for `s`.
theorem dsa_verify_inverse_of_verify(
    p: Nat,
    q: Nat,
    g: Nat,
    y: Nat,
    h: Nat,
    r: Nat,
    s: Nat,
    sinv: Nat
) {
    dsa_verify(p, q, g, y, h, r, s, sinv) implies is_mod_inverse(s, sinv, q)
} by {
    if dsa_verify(p, q, g, y, h, r, s, sinv) {
        dsa_verify_guards_of_verify(p, q, g, y, h, r, s, sinv)
        is_mod_inverse(s, sinv, q)
    }
}

/// Verification implies the final verifier equation.
theorem dsa_verify_core_equation_of_verify(
    p: Nat,
    q: Nat,
    g: Nat,
    y: Nat,
    h: Nat,
    r: Nat,
    s: Nat,
    sinv: Nat
) {
    dsa_verify(p, q, g, y, h, r, s, sinv) implies
        dsa_verify_core_equation(p, q, g, y, h, r, sinv)
} by {
    if dsa_verify(p, q, g, y, h, r, s, sinv) {
        dsa_verify(p, q, g, y, h, r, s, sinv) =
            (0 < r and r < q and 0 < s and s < q and
             is_mod_inverse(s, sinv, q) and
             (g.pow((h * sinv).mod(q)) * y.pow((r * sinv).mod(q))).mod(p).mod(q) = r)
        dsa_verify_core_equation(p, q, g, y, h, r, sinv)
    }
}

/// Verification implies that the subgroup-order modulus `q` is nonzero.
theorem dsa_verify_modulus_ne_zero(
    p: Nat,
    q: Nat,
    g: Nat,
    y: Nat,
    h: Nat,
    r: Nat,
    s: Nat,
    sinv: Nat
) {
    dsa_verify(p, q, g, y, h, r, s, sinv) implies q != 0
} by {
    if dsa_verify(p, q, g, y, h, r, s, sinv) {
        dsa_verify_signature_components_in_range_of_verify(p, q, g, y, h, r, s, sinv)
        dsa_signature_components_in_range(q, r, s)
        dsa_signature_components_in_range_r_lt(q, r, s)
        r < q
        if q = 0 {
            r < 0
            not_lt_zero(r)
            false
        }
    }
}

/// Public guards together with the verifier equation reconstruct `dsa_verify`.
theorem dsa_verify_of_guards_and_core_equation(
    p: Nat,
    q: Nat,
    g: Nat,
    y: Nat,
    h: Nat,
    r: Nat,
    s: Nat,
    sinv: Nat
) {
    dsa_signature_components_in_range(q, r, s) and
        is_mod_inverse(s, sinv, q) and
        dsa_verify_core_equation(p, q, g, y, h, r, sinv)
    implies dsa_verify(p, q, g, y, h, r, s, sinv)
} by {
    if dsa_signature_components_in_range(q, r, s) and
        is_mod_inverse(s, sinv, q) and
        dsa_verify_core_equation(p, q, g, y, h, r, sinv) {
        dsa_signature_components_in_range(q, r, s) =
            (dsa_r_in_range(q, r) and dsa_s_in_range(q, s))
        dsa_r_in_range(q, r) = (0 < r and r < q)
        dsa_s_in_range(q, s) = (0 < s and s < q)
        dsa_verify(p, q, g, y, h, r, s, sinv)
    }
}
