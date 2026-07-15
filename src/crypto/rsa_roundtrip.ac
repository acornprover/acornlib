from crypto.rsa import Nat, rsa_encrypt, rsa_decrypt, is_rsa_key,
    is_rsa_key_unfold, rsa_collapse, rsa_pow_mod_pq, rsa_correctness
numerals Nat

/// RSA key validity is symmetric in the public and private exponents.
theorem rsa_key_symmetric(p: Nat, q: Nat, e: Nat, d: Nat) {
    is_rsa_key(p, q, e, d) implies is_rsa_key(p, q, d, e)
} by {
    if is_rsa_key(p, q, e, d) {
        is_rsa_key_unfold(p, q, e, d)
        is_rsa_key_unfold(p, q, d, e)
        (d * e).mod((p - 1) * (q - 1)) = (e * d).mod((p - 1) * (q - 1))
        is_rsa_key(p, q, d, e)
    }
}

/// Normalized RSA correctness: decrypting an encrypted message recovers the
/// message modulo the RSA modulus, with no boundedness hypothesis on `m`.
theorem rsa_decrypt_encrypt_mod(p: Nat, q: Nat, e: Nat, d: Nat, m: Nat) {
    is_rsa_key(p, q, e, d) implies
        rsa_decrypt(rsa_encrypt(m, e, p * q), d, p * q) = m.mod(p * q)
} by {
    if is_rsa_key(p, q, e, d) {
        rsa_collapse(m, e, d, p * q)
        rsa_pow_mod_pq(p, q, e, d, m)
        m.pow(e * d).mod(p * q) = m.mod(p * q)
        rsa_decrypt(rsa_encrypt(m, e, p * q), d, p * q) = m.mod(p * q)
    }
}

/// Bounded reverse RSA roundtrip: applying the private exponent and then the
/// public exponent recovers a bounded message.
theorem rsa_encrypt_decrypt_bounded(p: Nat, q: Nat, e: Nat, d: Nat, m: Nat) {
    is_rsa_key(p, q, e, d) and m < p * q implies
        rsa_encrypt(rsa_decrypt(m, d, p * q), e, p * q) = m
} by {
    if is_rsa_key(p, q, e, d) and m < p * q {
        rsa_key_symmetric(p, q, e, d)
        rsa_correctness(p, q, d, e, m)
        rsa_decrypt(rsa_encrypt(m, d, p * q), e, p * q) = m
        rsa_encrypt(rsa_decrypt(m, d, p * q), e, p * q) = m
    }
}

/// Normalized reverse RSA roundtrip: applying the private exponent and then
/// the public exponent recovers the input modulo the RSA modulus.
theorem rsa_encrypt_decrypt_mod(p: Nat, q: Nat, e: Nat, d: Nat, m: Nat) {
    is_rsa_key(p, q, e, d) implies
        rsa_encrypt(rsa_decrypt(m, d, p * q), e, p * q) = m.mod(p * q)
} by {
    if is_rsa_key(p, q, e, d) {
        rsa_key_symmetric(p, q, e, d)
        rsa_decrypt_encrypt_mod(p, q, d, e, m)
        rsa_decrypt(rsa_encrypt(m, d, p * q), e, p * q) = m.mod(p * q)
        rsa_encrypt(rsa_decrypt(m, d, p * q), e, p * q) = m.mod(p * q)
    }
}

/// Bidirectional normalized RSA roundtrip: both public-then-private and
/// private-then-public exponentiation recover the input modulo the RSA modulus.
theorem rsa_roundtrip_mod_pair(p: Nat, q: Nat, e: Nat, d: Nat, m: Nat) {
    is_rsa_key(p, q, e, d) implies
        rsa_decrypt(rsa_encrypt(m, e, p * q), d, p * q) = m.mod(p * q) and
        rsa_encrypt(rsa_decrypt(m, d, p * q), e, p * q) = m.mod(p * q)
} by {
    if is_rsa_key(p, q, e, d) {
        rsa_decrypt_encrypt_mod(p, q, e, d, m)
        rsa_encrypt_decrypt_mod(p, q, e, d, m)
    }
}
