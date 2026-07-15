from nat import Nat
from nat import add_mod, small_mod
from nat import exp_mul
from number_theory import congr_mod_pow, mod_congr_mod_self
from number_theory import coprime_of_distinct_primes
from number_theory import nat_congr_combine_coprime
from number_theory import rsa_pow_congr
numerals Nat

/// RSA encryption of a message `m` with public exponent `e` modulo `n`.
/// Computes `m^e mod n`.
define rsa_encrypt(m: Nat, e: Nat, n: Nat) -> Nat {
    m.pow(e).mod(n)
}

/// RSA decryption of a ciphertext `c` with private exponent `d` modulo `n`.
/// Computes `c^d mod n`.
define rsa_decrypt(c: Nat, d: Nat, n: Nat) -> Nat {
    c.pow(d).mod(n)
}

/// True if `(p, q, e, d)` is a valid RSA key tuple: `p` and `q` are distinct
/// primes and `e * d` is congruent to `1` modulo `(p - 1) * (q - 1)`.
define is_rsa_key(p: Nat, q: Nat, e: Nat, d: Nat) -> Bool {
    p.is_prime and q.is_prime and p != q and
    (e * d).mod((p - 1) * (q - 1)) = 1
}

/// Bridge: is_rsa_key unfolds to its conjuncts.
theorem is_rsa_key_unfold(p: Nat, q: Nat, e: Nat, d: Nat) {
    is_rsa_key(p, q, e, d) =
        (p.is_prime and q.is_prime and p != q
            and (e * d).mod((p - 1) * (q - 1)) = 1)
}

/// Helper: from `(e*d).mod((p-1)*(q-1)) = 1`, extract a quotient k with
/// `k * ((p - 1) * (q - 1)) + 1 = e * d`.
theorem rsa_quotient_form(p: Nat, q: Nat, e: Nat, d: Nat) {
    (e * d).mod((p - 1) * (q - 1)) = 1
        implies exists(k: Nat) { k * ((p - 1) * (q - 1)) + 1 = e * d }
} by {
    if (e * d).mod((p - 1) * (q - 1)) = 1 {
        add_mod(e * d, (p - 1) * (q - 1))
        let k: Nat satisfy {
            k * ((p - 1) * (q - 1)) + (e * d).mod((p - 1) * (q - 1)) = e * d
        }
        k * ((p - 1) * (q - 1)) + 1 = e * d
    }
}

/// Helper: rewrite k * ((p-1)*(q-1)) as (k*(q-1)) * (p-1).
theorem rsa_assoc_for_p(k: Nat, p: Nat, q: Nat) {
    k * ((p - 1) * (q - 1)) = k * (q - 1) * (p - 1)
} by {
}

/// Helper: rewrite k * ((p-1)*(q-1)) as (k*(p-1)) * (q-1).
theorem rsa_assoc_for_q(k: Nat, p: Nat, q: Nat) {
    k * ((p - 1) * (q - 1)) = k * (p - 1) * (q - 1)
}

/// Power exponent matches the RSA pattern modulo p: m.pow(e*d) ≡ m (mod p).
theorem rsa_pow_mod_p(p: Nat, q: Nat, e: Nat, d: Nat, m: Nat, k: Nat) {
    p.is_prime and k * ((p - 1) * (q - 1)) + 1 = e * d
        implies m.pow(e * d).congr_mod(m, p)
} by {
    if p.is_prime and k * ((p - 1) * (q - 1)) + 1 = e * d {
        rsa_assoc_for_p(k, p, q)
        rsa_pow_congr(p, m, k * (q - 1))
        m.pow(e * d).congr_mod(m, p)
    }
}

/// Power exponent matches the RSA pattern modulo q: m.pow(e*d) ≡ m (mod q).
theorem rsa_pow_mod_q(p: Nat, q: Nat, e: Nat, d: Nat, m: Nat, k: Nat) {
    q.is_prime and k * ((p - 1) * (q - 1)) + 1 = e * d
        implies m.pow(e * d).congr_mod(m, q)
} by {
    if q.is_prime and k * ((p - 1) * (q - 1)) + 1 = e * d {
        rsa_assoc_for_q(k, p, q)
        rsa_pow_congr(q, m, k * (p - 1))
        m.pow(e * d).congr_mod(m, q)
    }
}

/// Combined power congruence: m.pow(e*d) ≡ m (mod p*q) for a valid RSA key.
theorem rsa_pow_mod_pq(p: Nat, q: Nat, e: Nat, d: Nat, m: Nat) {
    is_rsa_key(p, q, e, d) implies m.pow(e * d).congr_mod(m, p * q)
} by {
    if is_rsa_key(p, q, e, d) {
        is_rsa_key_unfold(p, q, e, d)
        // Unfold is_rsa_key.
        rsa_quotient_form(p, q, e, d)
        let k: Nat satisfy { k * ((p - 1) * (q - 1)) + 1 = e * d }
        rsa_pow_mod_p(p, q, e, d, m, k)
        rsa_pow_mod_q(p, q, e, d, m, k)
        m.pow(e * d).congr_mod(m, q)
        coprime_of_distinct_primes(p, q)
        nat_congr_combine_coprime(p, q, m.pow(e * d), m)
        m.pow(e * d).congr_mod(m, p * q)
    }
}

/// Helper: ((m^e mod n)^d) mod n equals m^(e*d) mod n via mod-pow compatibility.
theorem rsa_collapse(m: Nat, e: Nat, d: Nat, n: Nat) {
    ((m.pow(e).mod(n)).pow(d)).mod(n) = m.pow(e * d).mod(n)
} by {
    mod_congr_mod_self(m.pow(e), n)
    congr_mod_pow(m.pow(e).mod(n), m.pow(e), n, d)
    exp_mul(m, e, d)
}

/// RSA correctness: decrypting an encrypted message recovers the original.
/// For a valid RSA key `(p, q, e, d)` with modulus `n = p * q` and any
/// message `m < n`, we have `rsa_decrypt(rsa_encrypt(m, e, n), d, n) = m`.
theorem rsa_correctness(p: Nat, q: Nat, e: Nat, d: Nat, m: Nat) {
    is_rsa_key(p, q, e, d) and m < p * q implies
        rsa_decrypt(rsa_encrypt(m, e, p * q), d, p * q) = m
} by {
    if is_rsa_key(p, q, e, d) and m < p * q {
        // Unfold rsa_encrypt and rsa_decrypt to a power expression mod p*q.
        rsa_collapse(m, e, d, p * q)
        // Apply combined congruence and small_mod.
        rsa_pow_mod_pq(p, q, e, d, m)
        small_mod(m, p * q)
        m.pow(e * d).mod(p * q) = m
        rsa_decrypt(rsa_encrypt(m, e, p * q), d, p * q) = m
    }
}
