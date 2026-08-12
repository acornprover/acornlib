from crypto.rsa import Nat, rsa_encrypt, rsa_decrypt, is_rsa_key
from crypto.rsa_roundtrip import rsa_decrypt_encrypt_mod, rsa_encrypt_decrypt_mod
from number_theory import mod_congr_mod_self

numerals Nat

/// Public-then-private RSA roundtrip recovers any congruent representative modulo the RSA modulus.
theorem rsa_decrypt_encrypt_eq_of_congr_mod(p: Nat, q: Nat, e: Nat, d: Nat, m: Nat, r: Nat) {
    is_rsa_key(p, q, e, d) and m.congr_mod(r, p * q) implies
        rsa_decrypt(rsa_encrypt(m, e, p * q), d, p * q) = r.mod(p * q)
} by {
    if is_rsa_key(p, q, e, d) and m.congr_mod(r, p * q) {
        rsa_decrypt_encrypt_mod(p, q, e, d, m)
        rsa_decrypt(rsa_encrypt(m, e, p * q), d, p * q) = m.mod(p * q)
        m.mod(p * q) = r.mod(p * q)
        rsa_decrypt(rsa_encrypt(m, e, p * q), d, p * q) = r.mod(p * q)
    }
}

/// Private-then-public RSA roundtrip recovers any congruent representative modulo the RSA modulus.
theorem rsa_encrypt_decrypt_eq_of_congr_mod(p: Nat, q: Nat, e: Nat, d: Nat, m: Nat, r: Nat) {
    is_rsa_key(p, q, e, d) and m.congr_mod(r, p * q) implies
        rsa_encrypt(rsa_decrypt(m, d, p * q), e, p * q) = r.mod(p * q)
} by {
    if is_rsa_key(p, q, e, d) and m.congr_mod(r, p * q) {
        rsa_encrypt_decrypt_mod(p, q, e, d, m)
        rsa_encrypt(rsa_decrypt(m, d, p * q), e, p * q) = m.mod(p * q)
        m.mod(p * q) = r.mod(p * q)
        rsa_encrypt(rsa_decrypt(m, d, p * q), e, p * q) = r.mod(p * q)
    }
}

/// Bidirectional normalized RSA roundtrip with an arbitrary congruent representative.
theorem rsa_roundtrip_mod_pair_of_congr_mod(p: Nat, q: Nat, e: Nat, d: Nat, m: Nat, r: Nat) {
    is_rsa_key(p, q, e, d) and m.congr_mod(r, p * q) implies
        rsa_decrypt(rsa_encrypt(m, e, p * q), d, p * q) = r.mod(p * q) and
        rsa_encrypt(rsa_decrypt(m, d, p * q), e, p * q) = r.mod(p * q)
} by {
    if is_rsa_key(p, q, e, d) and m.congr_mod(r, p * q) {
        rsa_decrypt_encrypt_eq_of_congr_mod(p, q, e, d, m, r)
        rsa_encrypt_decrypt_eq_of_congr_mod(p, q, e, d, m, r)
        rsa_decrypt(rsa_encrypt(m, e, p * q), d, p * q) = r.mod(p * q)
        rsa_encrypt(rsa_decrypt(m, d, p * q), e, p * q) = r.mod(p * q)
        rsa_decrypt(rsa_encrypt(m, e, p * q), d, p * q) = r.mod(p * q) and
            rsa_encrypt(rsa_decrypt(m, d, p * q), e, p * q) = r.mod(p * q)
    }
}

/// Public-then-private output is congruent to any representative congruent to the input.
theorem rsa_decrypt_encrypt_congr_mod_of_congr_mod(p: Nat, q: Nat, e: Nat, d: Nat, m: Nat, r: Nat) {
    is_rsa_key(p, q, e, d) and m.congr_mod(r, p * q) implies
        rsa_decrypt(rsa_encrypt(m, e, p * q), d, p * q).congr_mod(r, p * q)
} by {
    if is_rsa_key(p, q, e, d) and m.congr_mod(r, p * q) {
        rsa_decrypt_encrypt_eq_of_congr_mod(p, q, e, d, m, r)
        rsa_decrypt(rsa_encrypt(m, e, p * q), d, p * q) = r.mod(p * q)
        mod_congr_mod_self(r, p * q)
        r.mod(p * q).congr_mod(r, p * q)
        rsa_decrypt(rsa_encrypt(m, e, p * q), d, p * q).congr_mod(r, p * q)
    }
}

/// Private-then-public output is congruent to any representative congruent to the input.
theorem rsa_encrypt_decrypt_congr_mod_of_congr_mod(p: Nat, q: Nat, e: Nat, d: Nat, m: Nat, r: Nat) {
    is_rsa_key(p, q, e, d) and m.congr_mod(r, p * q) implies
        rsa_encrypt(rsa_decrypt(m, d, p * q), e, p * q).congr_mod(r, p * q)
} by {
    if is_rsa_key(p, q, e, d) and m.congr_mod(r, p * q) {
        rsa_encrypt_decrypt_eq_of_congr_mod(p, q, e, d, m, r)
        rsa_encrypt(rsa_decrypt(m, d, p * q), e, p * q) = r.mod(p * q)
        mod_congr_mod_self(r, p * q)
        r.mod(p * q).congr_mod(r, p * q)
        rsa_encrypt(rsa_decrypt(m, d, p * q), e, p * q).congr_mod(r, p * q)
    }
}
