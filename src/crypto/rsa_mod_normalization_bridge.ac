from crypto.rsa import Nat, is_rsa_key, rsa_encrypt, rsa_decrypt,
    rsa_correctness
from crypto.rsa_roundtrip import rsa_key_symmetric, rsa_roundtrip_mod_pair,
    rsa_encrypt_decrypt_bounded
numerals Nat

/// RSA key validity is invariant under swapping the public and private exponents.
theorem rsa_key_symmetric_iff(p: Nat, q: Nat, e: Nat, d: Nat) {
    is_rsa_key(p, q, e, d) = is_rsa_key(p, q, d, e)
} by {
    if is_rsa_key(p, q, e, d) {
        rsa_key_symmetric(p, q, e, d)
        is_rsa_key(p, q, d, e)
    }
    if is_rsa_key(p, q, d, e) {
        rsa_key_symmetric(p, q, d, e)
        is_rsa_key(p, q, e, d)
    }
}

/// For a bounded message, both RSA exponent orders recover the message.
theorem rsa_roundtrip_bounded_pair(p: Nat, q: Nat, e: Nat, d: Nat, m: Nat) {
    is_rsa_key(p, q, e, d) and m < p * q implies
        rsa_decrypt(rsa_encrypt(m, e, p * q), d, p * q) = m and
        rsa_encrypt(rsa_decrypt(m, d, p * q), e, p * q) = m
} by {
    if is_rsa_key(p, q, e, d) and m < p * q {
        rsa_correctness(p, q, e, d, m)
        rsa_decrypt(rsa_encrypt(m, e, p * q), d, p * q) = m
        rsa_encrypt_decrypt_bounded(p, q, e, d, m)
        rsa_encrypt(rsa_decrypt(m, d, p * q), e, p * q) = m
        rsa_decrypt(rsa_encrypt(m, e, p * q), d, p * q) = m and
            rsa_encrypt(rsa_decrypt(m, d, p * q), e, p * q) = m
    }
}

/// The normalized bidirectional RSA roundtrip is preserved when the exponents are swapped.
theorem rsa_roundtrip_mod_pair_symmetric(p: Nat, q: Nat, e: Nat, d: Nat, m: Nat) {
    is_rsa_key(p, q, e, d) implies
        rsa_decrypt(rsa_encrypt(m, d, p * q), e, p * q) = m.mod(p * q) and
        rsa_encrypt(rsa_decrypt(m, e, p * q), d, p * q) = m.mod(p * q)
} by {
    if is_rsa_key(p, q, e, d) {
        rsa_key_symmetric(p, q, e, d)
        is_rsa_key(p, q, d, e)
        rsa_roundtrip_mod_pair(p, q, d, e, m)
        rsa_decrypt(rsa_encrypt(m, d, p * q), e, p * q) = m.mod(p * q)
        rsa_encrypt(rsa_decrypt(m, e, p * q), d, p * q) = m.mod(p * q)
        rsa_decrypt(rsa_encrypt(m, d, p * q), e, p * q) = m.mod(p * q) and
            rsa_encrypt(rsa_decrypt(m, e, p * q), d, p * q) = m.mod(p * q)
    }
}
