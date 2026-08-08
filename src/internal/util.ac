/// Compatibility shim re-exporting function and relation infrastructure from narrower modules.

from data.basic.functions import Inhabited, binary_function_eq_apply, binary_function_extensionality,
    binary_function_eq_transport_predicate, binary_function_eq_transport_predicate_rev,
    compose, compose_eq, compose_eq_left, compose_eq_right, eq_transport_predicate,
    eq_transport_predicate_rev, flip, function_eq_apply, function_eq_transport_bijection,
    function_eq_transport_bijection_fn, function_eq_transport_bijection_fn_rev,
    function_eq_transport_bijection_rev, function_eq_transport_constant,
    function_eq_transport_constant_rev, function_eq_transport_injective,
    function_eq_transport_injective_fn, function_eq_transport_injective_fn_rev,
    function_eq_transport_injective_rev, function_eq_transport_predicate,
    function_eq_transport_predicate_rev, function_eq_transport_surjective,
    function_eq_transport_surjective_fn, function_eq_transport_surjective_fn_rev,
    function_eq_transport_surjective_rev, function_extensionality, injective_fn_eq,
    is_bijection, is_bijection_eq_is_bijection_fn, is_bijection_fn, is_constant,
    is_injective, is_injective_eq_is_injective_fn, is_injective_fn, is_surjective,
    is_surjective_eq_is_surjective_fn, is_surjective_fn, predicate_eq_apply,
    predicate_eq_backward, predicate_eq_forward, predicate_eq_transport_predicate,
    predicate_eq_transport_predicate_rev, bijection_fn_is_injective, bijection_fn_is_surjective,
    surjective_fn_has_preimage
from data.basic.logic import and_assoc, and_comm, and_intro, and_left, and_or_distrib_left,
    and_right, contrapose, contrapose_iff, eq_false_elim, eq_false_intro,
    eq_true_elim, eq_true_intro, exists_intro, not_intro, not_not_elim, not_not_intro,
    or_and_distrib_left, or_assoc, or_comm, or_intro_left, or_intro_right
from data.basic.relation_basic import antisymmetric_eq, eq_relation, equivalence_flip,
    equivalence_imp_relation_converse_eq, equivalence_self, equivalence_step,
    is_antisymmetric, is_asymmetric, is_equivalence, is_irreflexive,
    is_partial_equivalence, is_reflexive, is_symmetric, is_total, is_transitive,
    partial_equivalence_flip, partial_equivalence_is_transitive, reflexive_self,
    relation_compose, relation_compose_assoc, relation_compose_eq_relation_left,
    relation_compose_eq_relation_right, relation_compose_has_witness,
    relation_compose_intro, relation_converse, relation_converse_compose,
    relation_converse_eq_relation, relation_converse_union, relation_eq_iff_subset_both,
    relation_eq_imp_antisymmetric,
    relation_eq_imp_antisymmetric_rev, relation_eq_imp_asymmetric,
    relation_eq_imp_asymmetric_rev, relation_eq_imp_equivalence,
    relation_eq_imp_equivalence_rev, relation_eq_imp_irreflexive,
    relation_eq_imp_irreflexive_rev, relation_eq_imp_partial_equivalence,
    relation_eq_imp_partial_equivalence_rev, relation_eq_imp_reflexive,
    relation_eq_imp_reflexive_rev, relation_eq_imp_symmetric, relation_eq_imp_symmetric_rev,
    relation_eq_imp_total, relation_eq_imp_total_rev, relation_eq_imp_transitive,
    relation_eq_imp_transitive_rev, relation_intersection, relation_intersection_assoc,
    relation_intersection_comm, relation_intersection_idempotent,
    relation_intersection_is_reflexive, relation_intersection_is_symmetric,
    relation_intersection_is_transitive, relation_intersection_monotone,
    relation_intersection_subset_left_eq, relation_intersection_subset_right_eq,
    relation_intersection_union_absorb, relation_intersection_union_distrib, relation_union,
    relation_union_assoc, relation_union_comm, relation_union_idempotent,
    relation_union_intersection_absorb, relation_union_intersection_distrib,
    relation_union_is_irreflexive, relation_union_is_reflexive, relation_union_is_symmetric,
    relation_union_is_total, relation_union_left_subset, relation_union_monotone,
    relation_union_right_subset, relation_union_subset, relation_union_subset_left_eq,
    relation_union_subset_right_eq, relation_subset, relation_subset_eq,
    relation_subset_eq_left, relation_subset_eq_right, relation_subset_intersection,
    relation_subset_refl, relation_subset_step, symmetric_flip, transitive_step
from data.basic.relation_transport import relation_pullback, relation_pullback_eq_relation_of_injective,
    relation_pullback_is_antisymmetric_of_injective, relation_pullback_is_asymmetric,
    relation_pullback_is_irreflexive, relation_pullback_is_reflexive,
    relation_pullback_is_total, relation_pullback_is_transitive, binary_congruence_eq,
    binary_congruence_eq_op, binary_congruence_eq_relation,
    preserves_binary_op_eq_codomain_op, preserves_binary_op_eq_domain_op,
    preserves_binary_op_eq_function, preserves_unary_op_eq_codomain_op,
    preserves_unary_op_eq_domain_op, preserves_unary_op_eq_function,
    respects_binary_op_eq, respects_binary_op_eq_op, respects_binary_op_eq_relation,
    respects_equivalence, respects_equivalence_eq, respects_equivalence_eq_function,
    respects_equivalence_eq_source, respects_equivalence_eq_target,
    respects_function_eq_function, respects_function_eq_relation,
    respects_predicate_eq_predicate, respects_predicate_eq_relation, respects_unary_op_eq,
    respects_unary_op_eq_op, respects_unary_op_eq_relation, unary_congruence_eq,
    unary_congruence_eq_op, unary_congruence_eq_relation
