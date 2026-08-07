/// Equivalences of categories.

from category.category import Category
from category.functor import Functor, identity_functor, compose_functor,
    identity_functor_src_cat, identity_functor_dst_cat,
    identity_functor_obj_map, identity_functor_mor_map,
    compose_functor_src_cat, compose_functor_dst_cat,
    compose_functor_obj_map, compose_functor_mor_map, compose_functor_some,
    functor_ext
from category.natural_transformation import NaturalTransformation, identity_nat_trans,
    identity_nat_trans_src_functor, identity_nat_trans_dst_functor
from category.natural_isomorphism import is_natural_isomorphism_pair,
    is_natural_isomorphism_pair_src_eq, is_natural_isomorphism_pair_dst_eq,
    is_natural_isomorphism_pair_swap
from category.natural_isomorphism import identity_nat_trans_is_natural_isomorphism
from data.basic.functions import compose, identity_fn, compose_identity_left

/// True if the forward and backward functors have matching source/target categories.
define category_equivalence_cats_axiom[O1, M1, O2, M2](
    forward: Functor[O1, M1, O2, M2], backward: Functor[O2, M2, O1, M1]
) -> Bool {
    forward.src_cat = backward.dst_cat
    and forward.dst_cat = backward.src_cat
}

/// True if `unit_fwd` is a natural transformation from the source identity functor to `backward ∘ forward`.
define category_equivalence_unit_axiom[O1, M1, O2, M2](
    forward: Functor[O1, M1, O2, M2], backward: Functor[O2, M2, O1, M1],
    unit_fwd: NaturalTransformation[O1, M1, O1, M1]
) -> Bool {
    unit_fwd.src_functor = identity_functor(forward.src_cat)
    and compose_functor(backward, forward) = Option.some(unit_fwd.dst_functor)
}

/// True if `counit_fwd` is a natural transformation from `forward ∘ backward` to the target identity functor.
define category_equivalence_counit_axiom[O1, M1, O2, M2](
    forward: Functor[O1, M1, O2, M2], backward: Functor[O2, M2, O1, M1],
    counit_fwd: NaturalTransformation[O2, M2, O2, M2]
) -> Bool {
    compose_functor(forward, backward) = Option.some(counit_fwd.src_functor)
    and counit_fwd.dst_functor = identity_functor(forward.dst_cat)
}

/// True if the given data forms an equivalence of categories.
define is_category_equivalence[O1, M1, O2, M2](
    forward: Functor[O1, M1, O2, M2], backward: Functor[O2, M2, O1, M1],
    unit_fwd: NaturalTransformation[O1, M1, O1, M1],
    unit_bwd: NaturalTransformation[O1, M1, O1, M1],
    counit_fwd: NaturalTransformation[O2, M2, O2, M2],
    counit_bwd: NaturalTransformation[O2, M2, O2, M2]
) -> Bool {
    category_equivalence_cats_axiom(forward, backward)
    and category_equivalence_unit_axiom(forward, backward, unit_fwd)
    and is_natural_isomorphism_pair(unit_fwd, unit_bwd)
    and category_equivalence_counit_axiom(forward, backward, counit_fwd)
    and is_natural_isomorphism_pair(counit_fwd, counit_bwd)
}

/// An equivalence of categories: a pair of functors with unit/counit natural isomorphisms.
structure CategoryEquivalence[O1, M1, O2, M2] {
    /// The forward functor.
    forward: Functor[O1, M1, O2, M2]

    /// The backward functor.
    backward: Functor[O2, M2, O1, M1]

    /// The forward direction of the unit natural isomorphism `1_C ≅ G ∘ F`.
    unit_fwd: NaturalTransformation[O1, M1, O1, M1]

    /// The backward direction of the unit natural isomorphism `G ∘ F ≅ 1_C`.
    unit_bwd: NaturalTransformation[O1, M1, O1, M1]

    /// The forward direction of the counit natural isomorphism `F ∘ G ≅ 1_D`.
    counit_fwd: NaturalTransformation[O2, M2, O2, M2]

    /// The backward direction of the counit natural isomorphism `1_D ≅ F ∘ G`.
    counit_bwd: NaturalTransformation[O2, M2, O2, M2]
} constraint {
    is_category_equivalence(forward, backward, unit_fwd, unit_bwd, counit_fwd, counit_bwd)
}

/// Construction of a category equivalence remembers its forward functor.
theorem category_equivalence_new_forward[O1, M1, O2, M2](
    forward: Functor[O1, M1, O2, M2], backward: Functor[O2, M2, O1, M1],
    unit_fwd: NaturalTransformation[O1, M1, O1, M1],
    unit_bwd: NaturalTransformation[O1, M1, O1, M1],
    counit_fwd: NaturalTransformation[O2, M2, O2, M2],
    counit_bwd: NaturalTransformation[O2, M2, O2, M2],
    e: CategoryEquivalence[O1, M1, O2, M2]
) {
    CategoryEquivalence[O1, M1, O2, M2].new(forward, backward, unit_fwd, unit_bwd, counit_fwd, counit_bwd)
        = Option.some(e) implies e.forward = forward
}

/// Construction of a category equivalence remembers its backward functor.
theorem category_equivalence_new_backward[O1, M1, O2, M2](
    forward: Functor[O1, M1, O2, M2], backward: Functor[O2, M2, O1, M1],
    unit_fwd: NaturalTransformation[O1, M1, O1, M1],
    unit_bwd: NaturalTransformation[O1, M1, O1, M1],
    counit_fwd: NaturalTransformation[O2, M2, O2, M2],
    counit_bwd: NaturalTransformation[O2, M2, O2, M2],
    e: CategoryEquivalence[O1, M1, O2, M2]
) {
    CategoryEquivalence[O1, M1, O2, M2].new(forward, backward, unit_fwd, unit_bwd, counit_fwd, counit_bwd)
        = Option.some(e) implies e.backward = backward
}

/// Construction of a category equivalence remembers its forward unit.
theorem category_equivalence_new_unit_fwd[O1, M1, O2, M2](
    forward: Functor[O1, M1, O2, M2], backward: Functor[O2, M2, O1, M1],
    unit_fwd: NaturalTransformation[O1, M1, O1, M1],
    unit_bwd: NaturalTransformation[O1, M1, O1, M1],
    counit_fwd: NaturalTransformation[O2, M2, O2, M2],
    counit_bwd: NaturalTransformation[O2, M2, O2, M2],
    e: CategoryEquivalence[O1, M1, O2, M2]
) {
    CategoryEquivalence[O1, M1, O2, M2].new(forward, backward, unit_fwd, unit_bwd, counit_fwd, counit_bwd)
        = Option.some(e) implies e.unit_fwd = unit_fwd
}

/// Construction of a category equivalence remembers its backward unit.
theorem category_equivalence_new_unit_bwd[O1, M1, O2, M2](
    forward: Functor[O1, M1, O2, M2], backward: Functor[O2, M2, O1, M1],
    unit_fwd: NaturalTransformation[O1, M1, O1, M1],
    unit_bwd: NaturalTransformation[O1, M1, O1, M1],
    counit_fwd: NaturalTransformation[O2, M2, O2, M2],
    counit_bwd: NaturalTransformation[O2, M2, O2, M2],
    e: CategoryEquivalence[O1, M1, O2, M2]
) {
    CategoryEquivalence[O1, M1, O2, M2].new(forward, backward, unit_fwd, unit_bwd, counit_fwd, counit_bwd)
        = Option.some(e) implies e.unit_bwd = unit_bwd
}

/// Construction of a category equivalence remembers its forward counit.
theorem category_equivalence_new_counit_fwd[O1, M1, O2, M2](
    forward: Functor[O1, M1, O2, M2], backward: Functor[O2, M2, O1, M1],
    unit_fwd: NaturalTransformation[O1, M1, O1, M1],
    unit_bwd: NaturalTransformation[O1, M1, O1, M1],
    counit_fwd: NaturalTransformation[O2, M2, O2, M2],
    counit_bwd: NaturalTransformation[O2, M2, O2, M2],
    e: CategoryEquivalence[O1, M1, O2, M2]
) {
    CategoryEquivalence[O1, M1, O2, M2].new(forward, backward, unit_fwd, unit_bwd, counit_fwd, counit_bwd)
        = Option.some(e) implies e.counit_fwd = counit_fwd
}

/// Construction of a category equivalence remembers its backward counit.
theorem category_equivalence_new_counit_bwd[O1, M1, O2, M2](
    forward: Functor[O1, M1, O2, M2], backward: Functor[O2, M2, O1, M1],
    unit_fwd: NaturalTransformation[O1, M1, O1, M1],
    unit_bwd: NaturalTransformation[O1, M1, O1, M1],
    counit_fwd: NaturalTransformation[O2, M2, O2, M2],
    counit_bwd: NaturalTransformation[O2, M2, O2, M2],
    e: CategoryEquivalence[O1, M1, O2, M2]
) {
    CategoryEquivalence[O1, M1, O2, M2].new(forward, backward, unit_fwd, unit_bwd, counit_fwd, counit_bwd)
        = Option.some(e) implies e.counit_bwd = counit_bwd
}

/// A category equivalence's data forms an equivalence.
theorem category_equivalence_is_equivalence[O1, M1, O2, M2](e: CategoryEquivalence[O1, M1, O2, M2]) {
    is_category_equivalence(e.forward, e.backward, e.unit_fwd, e.unit_bwd, e.counit_fwd, e.counit_bwd)
} by {
    CategoryEquivalence[O1, M1, O2, M2].constraint(
        e.forward, e.backward, e.unit_fwd, e.unit_bwd, e.counit_fwd, e.counit_bwd)
}

/// The forward and backward functors of an equivalence have matching source/target categories.
theorem category_equivalence_cats[O1, M1, O2, M2](e: CategoryEquivalence[O1, M1, O2, M2]) {
    e.forward.src_cat = e.backward.dst_cat
    and e.forward.dst_cat = e.backward.src_cat
} by {
    category_equivalence_is_equivalence(e)
    is_category_equivalence(e.forward, e.backward, e.unit_fwd, e.unit_bwd, e.counit_fwd, e.counit_bwd) =
        (category_equivalence_cats_axiom(e.forward, e.backward)
        and category_equivalence_unit_axiom(e.forward, e.backward, e.unit_fwd)
        and is_natural_isomorphism_pair(e.unit_fwd, e.unit_bwd)
        and category_equivalence_counit_axiom(e.forward, e.backward, e.counit_fwd)
        and is_natural_isomorphism_pair(e.counit_fwd, e.counit_bwd))
    category_equivalence_cats_axiom(e.forward, e.backward) =
        (e.forward.src_cat = e.backward.dst_cat
        and e.forward.dst_cat = e.backward.src_cat)
}

/// The unit's source functor is the identity on the source category.
theorem category_equivalence_unit_src_functor[O1, M1, O2, M2](e: CategoryEquivalence[O1, M1, O2, M2]) {
    e.unit_fwd.src_functor = identity_functor(e.forward.src_cat)
} by {
    category_equivalence_is_equivalence(e)
    is_category_equivalence(e.forward, e.backward, e.unit_fwd, e.unit_bwd, e.counit_fwd, e.counit_bwd) =
        (category_equivalence_cats_axiom(e.forward, e.backward)
        and category_equivalence_unit_axiom(e.forward, e.backward, e.unit_fwd)
        and is_natural_isomorphism_pair(e.unit_fwd, e.unit_bwd)
        and category_equivalence_counit_axiom(e.forward, e.backward, e.counit_fwd)
        and is_natural_isomorphism_pair(e.counit_fwd, e.counit_bwd))
}

/// The unit's target functor is the composition of backward with forward.
theorem category_equivalence_unit_dst_functor[O1, M1, O2, M2](e: CategoryEquivalence[O1, M1, O2, M2]) {
    compose_functor(e.backward, e.forward) = Option.some(e.unit_fwd.dst_functor)
} by {
    category_equivalence_is_equivalence(e)
    is_category_equivalence(e.forward, e.backward, e.unit_fwd, e.unit_bwd, e.counit_fwd, e.counit_bwd) =
        (category_equivalence_cats_axiom(e.forward, e.backward)
        and category_equivalence_unit_axiom(e.forward, e.backward, e.unit_fwd)
        and is_natural_isomorphism_pair(e.unit_fwd, e.unit_bwd)
        and category_equivalence_counit_axiom(e.forward, e.backward, e.counit_fwd)
        and is_natural_isomorphism_pair(e.counit_fwd, e.counit_bwd))
}

/// The unit forward and backward natural transformations form an inverse pair.
theorem category_equivalence_unit_iso[O1, M1, O2, M2](e: CategoryEquivalence[O1, M1, O2, M2]) {
    is_natural_isomorphism_pair(e.unit_fwd, e.unit_bwd)
} by {
    category_equivalence_is_equivalence(e)
    is_category_equivalence(e.forward, e.backward, e.unit_fwd, e.unit_bwd, e.counit_fwd, e.counit_bwd) =
        (category_equivalence_cats_axiom(e.forward, e.backward)
        and category_equivalence_unit_axiom(e.forward, e.backward, e.unit_fwd)
        and is_natural_isomorphism_pair(e.unit_fwd, e.unit_bwd)
        and category_equivalence_counit_axiom(e.forward, e.backward, e.counit_fwd)
        and is_natural_isomorphism_pair(e.counit_fwd, e.counit_bwd))
}

/// The counit's source functor is the composition of forward with backward.
theorem category_equivalence_counit_src_functor[O1, M1, O2, M2](e: CategoryEquivalence[O1, M1, O2, M2]) {
    compose_functor(e.forward, e.backward) = Option.some(e.counit_fwd.src_functor)
} by {
    category_equivalence_is_equivalence(e)
    is_category_equivalence(e.forward, e.backward, e.unit_fwd, e.unit_bwd, e.counit_fwd, e.counit_bwd) =
        (category_equivalence_cats_axiom(e.forward, e.backward)
        and category_equivalence_unit_axiom(e.forward, e.backward, e.unit_fwd)
        and is_natural_isomorphism_pair(e.unit_fwd, e.unit_bwd)
        and category_equivalence_counit_axiom(e.forward, e.backward, e.counit_fwd)
        and is_natural_isomorphism_pair(e.counit_fwd, e.counit_bwd))
}

/// The counit's target functor is the identity on the target category.
theorem category_equivalence_counit_dst_functor[O1, M1, O2, M2](e: CategoryEquivalence[O1, M1, O2, M2]) {
    e.counit_fwd.dst_functor = identity_functor(e.forward.dst_cat)
} by {
    category_equivalence_is_equivalence(e)
    is_category_equivalence(e.forward, e.backward, e.unit_fwd, e.unit_bwd, e.counit_fwd, e.counit_bwd) =
        (category_equivalence_cats_axiom(e.forward, e.backward)
        and category_equivalence_unit_axiom(e.forward, e.backward, e.unit_fwd)
        and is_natural_isomorphism_pair(e.unit_fwd, e.unit_bwd)
        and category_equivalence_counit_axiom(e.forward, e.backward, e.counit_fwd)
        and is_natural_isomorphism_pair(e.counit_fwd, e.counit_bwd))
}

/// The counit forward and backward natural transformations form an inverse pair.
theorem category_equivalence_counit_iso[O1, M1, O2, M2](e: CategoryEquivalence[O1, M1, O2, M2]) {
    is_natural_isomorphism_pair(e.counit_fwd, e.counit_bwd)
} by {
    category_equivalence_is_equivalence(e)
}

/// The composition of `identity_functor(c)` with itself has source category `c`.
theorem identity_functor_self_compose_src_cat[O, M](c: Category[O, M], h: Functor[O, M, O, M]) {
    compose_functor(identity_functor(c), identity_functor(c)) = Option.some(h) implies h.src_cat = c
} by {
    if compose_functor(identity_functor(c), identity_functor(c)) = Option.some(h) {
        compose_functor_src_cat(identity_functor(c), identity_functor(c), h)
        h.src_cat = identity_functor(c).src_cat
        identity_functor_src_cat(c)
    }
}

/// The composition of `identity_functor(c)` with itself has target category `c`.
theorem identity_functor_self_compose_dst_cat[O, M](c: Category[O, M], h: Functor[O, M, O, M]) {
    compose_functor(identity_functor(c), identity_functor(c)) = Option.some(h) implies h.dst_cat = c
} by {
    if compose_functor(identity_functor(c), identity_functor(c)) = Option.some(h) {
        compose_functor_dst_cat(identity_functor(c), identity_functor(c), h)
        h.dst_cat = identity_functor(c).dst_cat
        identity_functor_dst_cat(c)
    }
}

/// The composition of `identity_functor(c)` with itself has identity object map.
theorem identity_functor_self_compose_obj_map[O, M](c: Category[O, M], h: Functor[O, M, O, M]) {
    compose_functor(identity_functor(c), identity_functor(c)) = Option.some(h) implies h.obj_map = identity_fn[O]
} by {
    if compose_functor(identity_functor(c), identity_functor(c)) = Option.some(h) {
        compose_functor_obj_map(identity_functor(c), identity_functor(c), h)
        identity_functor_obj_map(c)
        compose_identity_left(identity_fn[O])
        h.obj_map = identity_fn[O]
    }
}

/// The composition of `identity_functor(c)` with itself has identity morphism map.
theorem identity_functor_self_compose_mor_map[O, M](c: Category[O, M], h: Functor[O, M, O, M]) {
    compose_functor(identity_functor(c), identity_functor(c)) = Option.some(h) implies h.mor_map = identity_fn[M]
} by {
    if compose_functor(identity_functor(c), identity_functor(c)) = Option.some(h) {
        compose_functor_mor_map(identity_functor(c), identity_functor(c), h)
        identity_functor_mor_map(c)
        compose_identity_left(identity_fn[M])
        h.mor_map = identity_fn[M]
    }
}

/// The composition of `identity_functor(c)` with itself is the identity functor.
theorem identity_functor_self_compose_eq_identity[O, M](c: Category[O, M], h: Functor[O, M, O, M]) {
    compose_functor(identity_functor(c), identity_functor(c)) = Option.some(h)
    implies h = identity_functor(c)
} by {
    if compose_functor(identity_functor(c), identity_functor(c)) = Option.some(h) {
        identity_functor_self_compose_src_cat(c, h)
        identity_functor_src_cat(c)

        identity_functor_self_compose_dst_cat(c, h)
        identity_functor_dst_cat(c)
        h.dst_cat = identity_functor(c).dst_cat

        identity_functor_self_compose_obj_map(c, h)
        identity_functor_obj_map(c)
        h.obj_map = identity_functor(c).obj_map

        identity_functor_self_compose_mor_map(c, h)
        identity_functor_mor_map(c)
        h.mor_map = identity_functor(c).mor_map

        functor_ext(h, identity_functor(c))
        h = identity_functor(c)
    }
}

/// The composition of the identity functor with itself is the identity functor.
theorem identity_functor_self_compose_some[O, M](c: Category[O, M]) {
    compose_functor(identity_functor(c), identity_functor(c)) = Option.some(identity_functor(c))
} by {
    identity_functor_src_cat(c)
    identity_functor_dst_cat(c)
    compose_functor_some(identity_functor(c), identity_functor(c))
    let h: Functor[O, M, O, M] satisfy {
        compose_functor(identity_functor(c), identity_functor(c)) = Option.some(h)
    }
    identity_functor_self_compose_eq_identity(c, h)
}

/// The identity natural transformation is the unit for the identity self-equivalence.
theorem identity_category_equivalence_unit_axiom[O, M](c: Category[O, M]) {
    category_equivalence_unit_axiom(identity_functor(c), identity_functor(c),
        identity_nat_trans(identity_functor(c)))
} by {
    identity_nat_trans_src_functor(identity_functor(c))
    identity_functor_self_compose_some(c)
    identity_nat_trans_dst_functor(identity_functor(c))
    identity_functor_src_cat(c)
    identity_nat_trans(identity_functor(c)).src_functor = identity_functor(identity_functor(c).src_cat)
}

/// The identity natural transformation is the counit for the identity self-equivalence.
theorem identity_category_equivalence_counit_axiom[O, M](c: Category[O, M]) {
    category_equivalence_counit_axiom(identity_functor(c), identity_functor(c),
        identity_nat_trans(identity_functor(c)))
} by {
    identity_nat_trans_src_functor(identity_functor(c))
    identity_functor_self_compose_some(c)
    identity_nat_trans_dst_functor(identity_functor(c))
    identity_functor_dst_cat(c)
    identity_nat_trans(identity_functor(c)).dst_functor = identity_functor(identity_functor(c).dst_cat)
}

/// The identity functor with identity unit and counit forms a category equivalence.
theorem identity_category_equivalence_axiom[O, M](c: Category[O, M]) {
    is_category_equivalence(identity_functor(c), identity_functor(c),
        identity_nat_trans(identity_functor(c)), identity_nat_trans(identity_functor(c)),
        identity_nat_trans(identity_functor(c)), identity_nat_trans(identity_functor(c)))
} by {
    let f = identity_functor(c)
    let n = identity_nat_trans(f)
    identity_functor_src_cat(c)
    identity_functor_dst_cat(c)
    category_equivalence_cats_axiom(f, f)
    identity_category_equivalence_unit_axiom(c)
    identity_nat_trans_is_natural_isomorphism(f)
    is_natural_isomorphism_pair(n, n)
    identity_category_equivalence_counit_axiom(c)
    category_equivalence_counit_axiom(f, f, n)
}

/// The identity equivalence of a category.
let identity_category_equivalence[O, M](c: Category[O, M]) -> result: CategoryEquivalence[O, M, O, M] satisfy {
    CategoryEquivalence[O, M, O, M].new(identity_functor(c), identity_functor(c),
        identity_nat_trans(identity_functor(c)), identity_nat_trans(identity_functor(c)),
        identity_nat_trans(identity_functor(c)), identity_nat_trans(identity_functor(c))) = Option.some(result)
} by {
    identity_category_equivalence_axiom(c)
}

/// The identity category equivalence has the identity functor as its forward functor.
theorem identity_category_equivalence_forward[O, M](c: Category[O, M]) {
    identity_category_equivalence(c).forward = identity_functor(c)
} by {
    let e = identity_category_equivalence(c)
    CategoryEquivalence[O, M, O, M].new(identity_functor(c), identity_functor(c),
        identity_nat_trans(identity_functor(c)), identity_nat_trans(identity_functor(c)),
        identity_nat_trans(identity_functor(c)), identity_nat_trans(identity_functor(c))) = Option.some(e)
    category_equivalence_new_forward(identity_functor(c), identity_functor(c),
        identity_nat_trans(identity_functor(c)), identity_nat_trans(identity_functor(c)),
        identity_nat_trans(identity_functor(c)), identity_nat_trans(identity_functor(c)), e)
}

/// The identity category equivalence has the identity functor as its backward functor.
theorem identity_category_equivalence_backward[O, M](c: Category[O, M]) {
    identity_category_equivalence(c).backward = identity_functor(c)
} by {
    let e = identity_category_equivalence(c)
    CategoryEquivalence[O, M, O, M].new(identity_functor(c), identity_functor(c),
        identity_nat_trans(identity_functor(c)), identity_nat_trans(identity_functor(c)),
        identity_nat_trans(identity_functor(c)), identity_nat_trans(identity_functor(c))) = Option.some(e)
    category_equivalence_new_backward(identity_functor(c), identity_functor(c),
        identity_nat_trans(identity_functor(c)), identity_nat_trans(identity_functor(c)),
        identity_nat_trans(identity_functor(c)), identity_nat_trans(identity_functor(c)), e)
}

/// The identity category equivalence has the identity natural transformation as its forward unit.
theorem identity_category_equivalence_unit_fwd[O, M](c: Category[O, M]) {
    identity_category_equivalence(c).unit_fwd = identity_nat_trans(identity_functor(c))
} by {
    let e = identity_category_equivalence(c)
    CategoryEquivalence[O, M, O, M].new(identity_functor(c), identity_functor(c),
        identity_nat_trans(identity_functor(c)), identity_nat_trans(identity_functor(c)),
        identity_nat_trans(identity_functor(c)), identity_nat_trans(identity_functor(c))) = Option.some(e)
    category_equivalence_new_unit_fwd(identity_functor(c), identity_functor(c),
        identity_nat_trans(identity_functor(c)), identity_nat_trans(identity_functor(c)),
        identity_nat_trans(identity_functor(c)), identity_nat_trans(identity_functor(c)), e)
}

/// The identity category equivalence has the identity natural transformation as its backward unit.
theorem identity_category_equivalence_unit_bwd[O, M](c: Category[O, M]) {
    identity_category_equivalence(c).unit_bwd = identity_nat_trans(identity_functor(c))
} by {
    let e = identity_category_equivalence(c)
    CategoryEquivalence[O, M, O, M].new(identity_functor(c), identity_functor(c),
        identity_nat_trans(identity_functor(c)), identity_nat_trans(identity_functor(c)),
        identity_nat_trans(identity_functor(c)), identity_nat_trans(identity_functor(c))) = Option.some(e)
    category_equivalence_new_unit_bwd(identity_functor(c), identity_functor(c),
        identity_nat_trans(identity_functor(c)), identity_nat_trans(identity_functor(c)),
        identity_nat_trans(identity_functor(c)), identity_nat_trans(identity_functor(c)), e)
}

/// The identity category equivalence has the identity natural transformation as its forward counit.
theorem identity_category_equivalence_counit_fwd[O, M](c: Category[O, M]) {
    identity_category_equivalence(c).counit_fwd = identity_nat_trans(identity_functor(c))
} by {
    let e = identity_category_equivalence(c)
    CategoryEquivalence[O, M, O, M].new(identity_functor(c), identity_functor(c),
        identity_nat_trans(identity_functor(c)), identity_nat_trans(identity_functor(c)),
        identity_nat_trans(identity_functor(c)), identity_nat_trans(identity_functor(c))) = Option.some(e)
    category_equivalence_new_counit_fwd(identity_functor(c), identity_functor(c),
        identity_nat_trans(identity_functor(c)), identity_nat_trans(identity_functor(c)),
        identity_nat_trans(identity_functor(c)), identity_nat_trans(identity_functor(c)), e)
}

/// The identity category equivalence has the identity natural transformation as its backward counit.
theorem identity_category_equivalence_counit_bwd[O, M](c: Category[O, M]) {
    identity_category_equivalence(c).counit_bwd = identity_nat_trans(identity_functor(c))
} by {
    let e = identity_category_equivalence(c)
    CategoryEquivalence[O, M, O, M].new(identity_functor(c), identity_functor(c),
        identity_nat_trans(identity_functor(c)), identity_nat_trans(identity_functor(c)),
        identity_nat_trans(identity_functor(c)), identity_nat_trans(identity_functor(c))) = Option.some(e)
    category_equivalence_new_counit_bwd(identity_functor(c), identity_functor(c),
        identity_nat_trans(identity_functor(c)), identity_nat_trans(identity_functor(c)),
        identity_nat_trans(identity_functor(c)), identity_nat_trans(identity_functor(c)), e)
}

/// Swapped cats axiom: with forward and backward exchanged, the cats axiom still holds.
theorem swap_category_equivalence_cats_axiom[O1, M1, O2, M2](e: CategoryEquivalence[O1, M1, O2, M2]) {
    category_equivalence_cats_axiom(e.backward, e.forward)
} by {
    category_equivalence_cats(e)
}

/// Swapped unit axiom: in the swapped equivalence, `counit_bwd` plays the role of the unit.
theorem swap_category_equivalence_unit_axiom[O1, M1, O2, M2](e: CategoryEquivalence[O1, M1, O2, M2]) {
    category_equivalence_unit_axiom(e.backward, e.forward, e.counit_bwd)
} by {
    category_equivalence_counit_iso(e)
    is_natural_isomorphism_pair_src_eq(e.counit_fwd, e.counit_bwd)
    is_natural_isomorphism_pair_dst_eq(e.counit_fwd, e.counit_bwd)
    category_equivalence_counit_dst_functor(e)
    category_equivalence_cats(e)
    category_equivalence_counit_src_functor(e)
    compose_functor(e.forward, e.backward) = Option.some(e.counit_bwd.dst_functor)
}

/// Swapped counit axiom: in the swapped equivalence, `unit_bwd` plays the role of the counit.
theorem swap_category_equivalence_counit_axiom[O1, M1, O2, M2](e: CategoryEquivalence[O1, M1, O2, M2]) {
    category_equivalence_counit_axiom(e.backward, e.forward, e.unit_bwd)
} by {
    category_equivalence_unit_iso(e)
    is_natural_isomorphism_pair_src_eq(e.unit_fwd, e.unit_bwd)
    is_natural_isomorphism_pair_dst_eq(e.unit_fwd, e.unit_bwd)
    category_equivalence_unit_dst_functor(e)
    category_equivalence_unit_src_functor(e)
    category_equivalence_cats(e)
    e.unit_bwd.dst_functor = identity_functor(e.backward.dst_cat)
}

/// Swapping forward/backward in an equivalence yields equivalence data in the opposite direction.
theorem swap_category_equivalence_axiom[O1, M1, O2, M2](e: CategoryEquivalence[O1, M1, O2, M2]) {
    is_category_equivalence(e.backward, e.forward, e.counit_bwd, e.counit_fwd, e.unit_bwd, e.unit_fwd)
} by {
    swap_category_equivalence_cats_axiom(e)
    swap_category_equivalence_unit_axiom(e)
    category_equivalence_counit_iso(e)
    is_natural_isomorphism_pair_swap(e.counit_fwd, e.counit_bwd)
    swap_category_equivalence_counit_axiom(e)
    category_equivalence_unit_iso(e)
    is_natural_isomorphism_pair_swap(e.unit_fwd, e.unit_bwd)
}

/// The category equivalence with forward and backward functors swapped.
let swap_category_equivalence[O1, M1, O2, M2](e: CategoryEquivalence[O1, M1, O2, M2]) -> result: CategoryEquivalence[O2, M2, O1, M1] satisfy {
    CategoryEquivalence[O2, M2, O1, M1].new(
        e.backward, e.forward, e.counit_bwd, e.counit_fwd, e.unit_bwd, e.unit_fwd
    ) = Option.some(result)
} by {
    swap_category_equivalence_axiom(e)
}

/// The swap of an equivalence has the original backward functor as its forward functor.
theorem swap_category_equivalence_forward[O1, M1, O2, M2](e: CategoryEquivalence[O1, M1, O2, M2]) {
    swap_category_equivalence(e).forward = e.backward
} by {
    let s = swap_category_equivalence(e)
    CategoryEquivalence[O2, M2, O1, M1].new(
        e.backward, e.forward, e.counit_bwd, e.counit_fwd, e.unit_bwd, e.unit_fwd
    ) = Option.some(s)
    category_equivalence_new_forward(
        e.backward, e.forward, e.counit_bwd, e.counit_fwd, e.unit_bwd, e.unit_fwd, s)
}

/// The swap of an equivalence has the original forward functor as its backward functor.
theorem swap_category_equivalence_backward[O1, M1, O2, M2](e: CategoryEquivalence[O1, M1, O2, M2]) {
    swap_category_equivalence(e).backward = e.forward
} by {
    let s = swap_category_equivalence(e)
    CategoryEquivalence[O2, M2, O1, M1].new(
        e.backward, e.forward, e.counit_bwd, e.counit_fwd, e.unit_bwd, e.unit_fwd
    ) = Option.some(s)
    category_equivalence_new_backward(
        e.backward, e.forward, e.counit_bwd, e.counit_fwd, e.unit_bwd, e.unit_fwd, s)
}

/// The swap of an equivalence uses the original backward counit as its forward unit.
theorem swap_category_equivalence_unit_fwd[O1, M1, O2, M2](e: CategoryEquivalence[O1, M1, O2, M2]) {
    swap_category_equivalence(e).unit_fwd = e.counit_bwd
} by {
    let s = swap_category_equivalence(e)
    CategoryEquivalence[O2, M2, O1, M1].new(
        e.backward, e.forward, e.counit_bwd, e.counit_fwd, e.unit_bwd, e.unit_fwd
    ) = Option.some(s)
    category_equivalence_new_unit_fwd(
        e.backward, e.forward, e.counit_bwd, e.counit_fwd, e.unit_bwd, e.unit_fwd, s)
}

/// The swap of an equivalence uses the original forward counit as its backward unit.
theorem swap_category_equivalence_unit_bwd[O1, M1, O2, M2](e: CategoryEquivalence[O1, M1, O2, M2]) {
    swap_category_equivalence(e).unit_bwd = e.counit_fwd
} by {
    let s = swap_category_equivalence(e)
    CategoryEquivalence[O2, M2, O1, M1].new(
        e.backward, e.forward, e.counit_bwd, e.counit_fwd, e.unit_bwd, e.unit_fwd
    ) = Option.some(s)
    category_equivalence_new_unit_bwd(
        e.backward, e.forward, e.counit_bwd, e.counit_fwd, e.unit_bwd, e.unit_fwd, s)
}

/// The swap of an equivalence uses the original backward unit as its forward counit.
theorem swap_category_equivalence_counit_fwd[O1, M1, O2, M2](e: CategoryEquivalence[O1, M1, O2, M2]) {
    swap_category_equivalence(e).counit_fwd = e.unit_bwd
} by {
    let s = swap_category_equivalence(e)
    CategoryEquivalence[O2, M2, O1, M1].new(
        e.backward, e.forward, e.counit_bwd, e.counit_fwd, e.unit_bwd, e.unit_fwd
    ) = Option.some(s)
    category_equivalence_new_counit_fwd(
        e.backward, e.forward, e.counit_bwd, e.counit_fwd, e.unit_bwd, e.unit_fwd, s)
}

/// The swap of an equivalence uses the original forward unit as its backward counit.
theorem swap_category_equivalence_counit_bwd[O1, M1, O2, M2](e: CategoryEquivalence[O1, M1, O2, M2]) {
    swap_category_equivalence(e).counit_bwd = e.unit_fwd
} by {
    let s = swap_category_equivalence(e)
    CategoryEquivalence[O2, M2, O1, M1].new(
        e.backward, e.forward, e.counit_bwd, e.counit_fwd, e.unit_bwd, e.unit_fwd
    ) = Option.some(s)
    category_equivalence_new_counit_bwd(
        e.backward, e.forward, e.counit_bwd, e.counit_fwd, e.unit_bwd, e.unit_fwd, s)
}
