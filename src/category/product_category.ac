/// The product category of two categories.

from category.category import Category, is_category, category_id_endpoints_constraint,
    category_compose_src_constraint, category_compose_dst_constraint,
    category_assoc_constraint, category_id_left_constraint,
    category_id_right_constraint, category_identity_src, category_identity_dst,
    category_compose_src, category_compose_dst, category_compose_assoc,
    category_id_left, category_id_right, category_new_src, category_new_dst,
    category_new_identity, category_new_compose
from pair import Pair, pair_eta, pair_new_first, pair_new_second

/// The componentwise source map for the product category.
define product_src[O1, M1, O2, M2](
    c: Category[O1, M1],
    d: Category[O2, M2],
    p: Pair[M1, M2]
) -> Pair[O1, O2] {
    Pair.new(c.src(p.first), d.src(p.second))
}

/// The first projection of `product_src` is the source in the first category.
theorem product_src_first[O1, M1, O2, M2](
    c: Category[O1, M1], d: Category[O2, M2], p: Pair[M1, M2]
) {
    product_src(c, d, p).first = c.src(p.first)
}

/// The second projection of `product_src` is the source in the second category.
theorem product_src_second[O1, M1, O2, M2](
    c: Category[O1, M1], d: Category[O2, M2], p: Pair[M1, M2]
) {
    product_src(c, d, p).second = d.src(p.second)
}

/// The componentwise target map for the product category.
define product_dst[O1, M1, O2, M2](
    c: Category[O1, M1],
    d: Category[O2, M2],
    p: Pair[M1, M2]
) -> Pair[O1, O2] {
    Pair.new(c.dst(p.first), d.dst(p.second))
}

/// The first projection of `product_dst` is the target in the first category.
theorem product_dst_first[O1, M1, O2, M2](
    c: Category[O1, M1], d: Category[O2, M2], p: Pair[M1, M2]
) {
    product_dst(c, d, p).first = c.dst(p.first)
}

/// The second projection of `product_dst` is the target in the second category.
theorem product_dst_second[O1, M1, O2, M2](
    c: Category[O1, M1], d: Category[O2, M2], p: Pair[M1, M2]
) {
    product_dst(c, d, p).second = d.dst(p.second)
}

/// The componentwise identity-assigning map for the product category.
define product_identity[O1, M1, O2, M2](
    c: Category[O1, M1],
    d: Category[O2, M2],
    x: Pair[O1, O2]
) -> Pair[M1, M2] {
    Pair.new(c.identity(x.first), d.identity(x.second))
}

/// The first projection of `product_identity` is the identity in the first category.
theorem product_identity_first[O1, M1, O2, M2](
    c: Category[O1, M1], d: Category[O2, M2], x: Pair[O1, O2]
) {
    product_identity(c, d, x).first = c.identity(x.first)
}

/// The second projection of `product_identity` is the identity in the second category.
theorem product_identity_second[O1, M1, O2, M2](
    c: Category[O1, M1], d: Category[O2, M2], x: Pair[O1, O2]
) {
    product_identity(c, d, x).second = d.identity(x.second)
}

/// The componentwise composition operator for the product category.
define product_compose[O1, M1, O2, M2](
    c: Category[O1, M1],
    d: Category[O2, M2],
    f: Pair[M1, M2],
    g: Pair[M1, M2]
) -> Pair[M1, M2] {
    Pair.new(c.compose(f.first, g.first), d.compose(f.second, g.second))
}

/// The first projection of `product_compose` is the composition in the first category.
theorem product_compose_first[O1, M1, O2, M2](
    c: Category[O1, M1], d: Category[O2, M2], f: Pair[M1, M2], g: Pair[M1, M2]
) {
    product_compose(c, d, f, g).first = c.compose(f.first, g.first)
}

/// The second projection of `product_compose` is the composition in the second category.
theorem product_compose_second[O1, M1, O2, M2](
    c: Category[O1, M1], d: Category[O2, M2], f: Pair[M1, M2], g: Pair[M1, M2]
) {
    product_compose(c, d, f, g).second = d.compose(f.second, g.second)
}

/// In the product category, the source of the identity equals the underlying object.
theorem product_identity_src[O1, M1, O2, M2](
    c: Category[O1, M1], d: Category[O2, M2], x: Pair[O1, O2]
) {
    product_src(c, d, product_identity(c, d, x)) = x
} by {
    product_identity_first(c, d, x)
    category_identity_src(c, x.first)
    product_src_first(c, d, product_identity(c, d, x))
    product_src(c, d, product_identity(c, d, x)).first = x.first
    product_identity_second(c, d, x)
    category_identity_src(d, x.second)
    product_src_second(c, d, product_identity(c, d, x))
    product_src(c, d, product_identity(c, d, x)).second = x.second
    pair_eta(x)
}

/// In the product category, the target of the identity equals the underlying object.
theorem product_identity_dst[O1, M1, O2, M2](
    c: Category[O1, M1], d: Category[O2, M2], x: Pair[O1, O2]
) {
    product_dst(c, d, product_identity(c, d, x)) = x
} by {
    product_identity_first(c, d, x)
    category_identity_dst(c, x.first)
    product_dst_first(c, d, product_identity(c, d, x))
    product_dst(c, d, product_identity(c, d, x)).first = x.first
    product_identity_second(c, d, x)
    category_identity_dst(d, x.second)
    product_dst_second(c, d, product_identity(c, d, x))
    product_dst(c, d, product_identity(c, d, x)).second = x.second
    pair_eta(x)
}

/// The product data satisfies the identity-endpoints axiom.
theorem product_id_endpoints[O1, M1, O2, M2](c: Category[O1, M1], d: Category[O2, M2]) {
    category_id_endpoints_constraint(product_src(c, d), product_dst(c, d), product_identity(c, d))
} by {
    forall(x: Pair[O1, O2]) {
        product_identity_src(c, d, x)
        product_identity_dst(c, d, x)
        (product_src(c, d, product_identity(c, d, x)) = x and product_dst(c, d, product_identity(c, d, x)) = x)
    }
}

/// In the product category, composability matches componentwise composability of the first component.
theorem product_composable_first[O1, M1, O2, M2](
    c: Category[O1, M1], d: Category[O2, M2], f: Pair[M1, M2], g: Pair[M1, M2]
) {
    product_src(c, d, f) = product_dst(c, d, g) implies c.src(f.first) = c.dst(g.first)
} by {
    if product_src(c, d, f) = product_dst(c, d, g) {
        product_src_first(c, d, f)
        product_dst_first(c, d, g)
        product_src(c, d, f).first = product_dst(c, d, g).first
    }
}

/// In the product category, composability matches componentwise composability of the second component.
theorem product_composable_second[O1, M1, O2, M2](
    c: Category[O1, M1], d: Category[O2, M2], f: Pair[M1, M2], g: Pair[M1, M2]
) {
    product_src(c, d, f) = product_dst(c, d, g) implies d.src(f.second) = d.dst(g.second)
} by {
    if product_src(c, d, f) = product_dst(c, d, g) {
        product_src_second(c, d, f)
        product_dst_second(c, d, g)
        product_src(c, d, f).second = product_dst(c, d, g).second
    }
}

/// The product data satisfies the composition source axiom.
theorem product_compose_src[O1, M1, O2, M2](c: Category[O1, M1], d: Category[O2, M2]) {
    category_compose_src_constraint(product_src(c, d), product_dst(c, d), product_compose(c, d))
} by {
    forall(f: Pair[M1, M2], g: Pair[M1, M2]) {
        if product_src(c, d, f) = product_dst(c, d, g) {
            product_composable_first(c, d, f, g)
            product_composable_second(c, d, f, g)
            category_compose_src(c, f.first, g.first)
            c.src(c.compose(f.first, g.first)) = c.src(g.first)
            category_compose_src(d, f.second, g.second)
            d.src(d.compose(f.second, g.second)) = d.src(g.second)
            product_compose_first(c, d, f, g)
            product_src_first(c, d, product_compose(c, d, f, g))
            product_src(c, d, product_compose(c, d, f, g)).first = c.src(g.first)
            product_src_first(c, d, g)
            product_src(c, d, product_compose(c, d, f, g)).first = product_src(c, d, g).first
            product_compose_second(c, d, f, g)
            product_src_second(c, d, product_compose(c, d, f, g))
            product_src(c, d, product_compose(c, d, f, g)).second = d.src(g.second)
            product_src_second(c, d, g)
            product_src(c, d, product_compose(c, d, f, g)).second = product_src(c, d, g).second
            product_src(c, d, product_compose(c, d, f, g)) = product_src(c, d, g)
        }
    }
}

/// The product data satisfies the composition target axiom.
theorem product_compose_dst[O1, M1, O2, M2](c: Category[O1, M1], d: Category[O2, M2]) {
    category_compose_dst_constraint(product_src(c, d), product_dst(c, d), product_compose(c, d))
} by {
    forall(f: Pair[M1, M2], g: Pair[M1, M2]) {
        if product_src(c, d, f) = product_dst(c, d, g) {
            product_composable_first(c, d, f, g)
            product_composable_second(c, d, f, g)
            category_compose_dst(c, f.first, g.first)
            c.dst(c.compose(f.first, g.first)) = c.dst(f.first)
            category_compose_dst(d, f.second, g.second)
            d.dst(d.compose(f.second, g.second)) = d.dst(f.second)
            product_compose_first(c, d, f, g)
            product_dst_first(c, d, product_compose(c, d, f, g))
            product_dst(c, d, product_compose(c, d, f, g)).first = c.dst(f.first)
            product_dst_first(c, d, f)
            product_dst(c, d, product_compose(c, d, f, g)).first = product_dst(c, d, f).first
            product_compose_second(c, d, f, g)
            product_dst_second(c, d, product_compose(c, d, f, g))
            product_dst(c, d, product_compose(c, d, f, g)).second = d.dst(f.second)
            product_dst_second(c, d, f)
            product_dst(c, d, product_compose(c, d, f, g)).second = product_dst(c, d, f).second
            product_dst(c, d, product_compose(c, d, f, g)) = product_dst(c, d, f)
        }
    }
}

/// The product data satisfies the associativity axiom.
theorem product_assoc[O1, M1, O2, M2](c: Category[O1, M1], d: Category[O2, M2]) {
    category_assoc_constraint(product_src(c, d), product_dst(c, d), product_compose(c, d))
} by {
    forall(f: Pair[M1, M2], g: Pair[M1, M2], h: Pair[M1, M2]) {
        if product_src(c, d, f) = product_dst(c, d, g) and product_src(c, d, g) = product_dst(c, d, h) {
            product_composable_first(c, d, f, g)
            product_composable_second(c, d, f, g)
            product_composable_first(c, d, g, h)
            product_composable_second(c, d, g, h)
            category_compose_assoc(c, f.first, g.first, h.first)
            let lhs1 = c.compose(c.compose(f.first, g.first), h.first)
            let rhs1 = c.compose(f.first, c.compose(g.first, h.first))
            lhs1 = rhs1
            category_compose_assoc(d, f.second, g.second, h.second)
            let lhs2 = d.compose(d.compose(f.second, g.second), h.second)
            let rhs2 = d.compose(f.second, d.compose(g.second, h.second))
            lhs2 = rhs2
            product_compose_first(c, d, f, g)
            product_compose_first(c, d, product_compose(c, d, f, g), h)
            product_compose(c, d, product_compose(c, d, f, g), h).first = lhs1
            product_compose_first(c, d, g, h)
            product_compose_first(c, d, f, product_compose(c, d, g, h))
            product_compose(c, d, f, product_compose(c, d, g, h)).first = rhs1
            product_compose(c, d, product_compose(c, d, f, g), h).first = product_compose(c, d, f, product_compose(c, d, g, h)).first
            product_compose_second(c, d, f, g)
            product_compose_second(c, d, product_compose(c, d, f, g), h)
            product_compose(c, d, product_compose(c, d, f, g), h).second = lhs2
            product_compose_second(c, d, g, h)
            product_compose_second(c, d, f, product_compose(c, d, g, h))
            product_compose(c, d, f, product_compose(c, d, g, h)).second = rhs2
            product_compose(c, d, product_compose(c, d, f, g), h).second = product_compose(c, d, f, product_compose(c, d, g, h)).second
            product_compose(c, d, product_compose(c, d, f, g), h) = product_compose(c, d, f, product_compose(c, d, g, h))
        }
    }
}

/// The product data satisfies the left-identity axiom.
theorem product_id_left[O1, M1, O2, M2](c: Category[O1, M1], d: Category[O2, M2]) {
    category_id_left_constraint(product_src(c, d), product_dst(c, d), product_identity(c, d), product_compose(c, d))
} by {
    forall(f: Pair[M1, M2]) {
        product_dst_first(c, d, f)
        product_identity_first(c, d, product_dst(c, d, f))
        product_identity(c, d, product_dst(c, d, f)).first = c.identity(c.dst(f.first))
        product_dst_second(c, d, f)
        product_identity_second(c, d, product_dst(c, d, f))
        product_identity(c, d, product_dst(c, d, f)).second = d.identity(d.dst(f.second))
        category_id_left(c, f.first)
        c.compose(c.identity(c.dst(f.first)), f.first) = f.first
        category_id_left(d, f.second)
        d.compose(d.identity(d.dst(f.second)), f.second) = f.second
        product_compose_first(c, d, product_identity(c, d, product_dst(c, d, f)), f)
        product_compose(c, d, product_identity(c, d, product_dst(c, d, f)), f).first = c.compose(product_identity(c, d, product_dst(c, d, f)).first, f.first)
        product_compose(c, d, product_identity(c, d, product_dst(c, d, f)), f).first = c.compose(c.identity(c.dst(f.first)), f.first)
        product_compose(c, d, product_identity(c, d, product_dst(c, d, f)), f).first = f.first
        product_compose_second(c, d, product_identity(c, d, product_dst(c, d, f)), f)
        product_compose(c, d, product_identity(c, d, product_dst(c, d, f)), f).second = d.compose(product_identity(c, d, product_dst(c, d, f)).second, f.second)
        product_compose(c, d, product_identity(c, d, product_dst(c, d, f)), f).second = d.compose(d.identity(d.dst(f.second)), f.second)
        product_compose(c, d, product_identity(c, d, product_dst(c, d, f)), f).second = f.second
        pair_eta(f)
        product_compose(c, d, product_identity(c, d, product_dst(c, d, f)), f) = f
    }
}

/// The product data satisfies the right-identity axiom.
theorem product_id_right[O1, M1, O2, M2](c: Category[O1, M1], d: Category[O2, M2]) {
    category_id_right_constraint(product_src(c, d), product_dst(c, d), product_identity(c, d), product_compose(c, d))
} by {
    forall(f: Pair[M1, M2]) {
        product_src_first(c, d, f)
        product_identity_first(c, d, product_src(c, d, f))
        product_identity(c, d, product_src(c, d, f)).first = c.identity(c.src(f.first))
        product_src_second(c, d, f)
        product_identity_second(c, d, product_src(c, d, f))
        product_identity(c, d, product_src(c, d, f)).second = d.identity(d.src(f.second))
        category_id_right(c, f.first)
        c.compose(f.first, c.identity(c.src(f.first))) = f.first
        category_id_right(d, f.second)
        d.compose(f.second, d.identity(d.src(f.second))) = f.second
        product_compose_first(c, d, f, product_identity(c, d, product_src(c, d, f)))
        product_compose(c, d, f, product_identity(c, d, product_src(c, d, f))).first = c.compose(f.first, product_identity(c, d, product_src(c, d, f)).first)
        product_compose(c, d, f, product_identity(c, d, product_src(c, d, f))).first = c.compose(f.first, c.identity(c.src(f.first)))
        product_compose(c, d, f, product_identity(c, d, product_src(c, d, f))).first = f.first
        product_compose_second(c, d, f, product_identity(c, d, product_src(c, d, f)))
        product_compose(c, d, f, product_identity(c, d, product_src(c, d, f))).second = d.compose(f.second, product_identity(c, d, product_src(c, d, f)).second)
        product_compose(c, d, f, product_identity(c, d, product_src(c, d, f))).second = d.compose(f.second, d.identity(d.src(f.second)))
        product_compose(c, d, f, product_identity(c, d, product_src(c, d, f))).second = f.second
        pair_eta(f)
        product_compose(c, d, f, product_identity(c, d, product_src(c, d, f))) = f
    }
}

/// The proposed product data satisfies the category axioms.
theorem product_is_category[O1, M1, O2, M2](c: Category[O1, M1], d: Category[O2, M2]) {
    is_category[Pair[O1, O2], Pair[M1, M2]](
        product_src(c, d), product_dst(c, d),
        product_identity(c, d), product_compose(c, d)
    )
} by {
    product_id_endpoints(c, d)
    product_compose_src(c, d)
    product_compose_dst(c, d)
    product_assoc(c, d)
    product_id_left(c, d)
    product_id_right(c, d)
    (category_id_endpoints_constraint(product_src(c, d), product_dst(c, d), product_identity(c, d))
        and category_compose_src_constraint(product_src(c, d), product_dst(c, d), product_compose(c, d))
        and category_compose_dst_constraint(product_src(c, d), product_dst(c, d), product_compose(c, d))
        and category_assoc_constraint(product_src(c, d), product_dst(c, d), product_compose(c, d))
        and category_id_left_constraint(product_src(c, d), product_dst(c, d), product_identity(c, d), product_compose(c, d))
        and category_id_right_constraint(product_src(c, d), product_dst(c, d), product_identity(c, d), product_compose(c, d)))
}

/// The product category of two categories.
let product_category[O1, M1, O2, M2](c: Category[O1, M1], d: Category[O2, M2]) -> result: Category[Pair[O1, O2], Pair[M1, M2]] satisfy {
    Category[Pair[O1, O2], Pair[M1, M2]].new(product_src(c, d), product_dst(c, d), product_identity(c, d), product_compose(c, d)) = Option.some(result)
} by {
    product_is_category(c, d)
}

/// The source map of the product category is `product_src`.
theorem product_category_src[O1, M1, O2, M2](c: Category[O1, M1], d: Category[O2, M2]) {
    product_category(c, d).src = product_src(c, d)
} by {
    let p = product_category(c, d)
    (Category[Pair[O1, O2], Pair[M1, M2]].new(product_src(c, d), product_dst(c, d), product_identity(c, d), product_compose(c, d)) = Option.some(p))
    category_new_src(product_src(c, d), product_dst(c, d), product_identity(c, d), product_compose(c, d), p)
}

/// The target map of the product category is `product_dst`.
theorem product_category_dst[O1, M1, O2, M2](c: Category[O1, M1], d: Category[O2, M2]) {
    product_category(c, d).dst = product_dst(c, d)
} by {
    let p = product_category(c, d)
    (Category[Pair[O1, O2], Pair[M1, M2]].new(product_src(c, d), product_dst(c, d), product_identity(c, d), product_compose(c, d)) = Option.some(p))
    category_new_dst(product_src(c, d), product_dst(c, d), product_identity(c, d), product_compose(c, d), p)
}

/// The identity-assigning map of the product category is `product_identity`.
theorem product_category_identity[O1, M1, O2, M2](c: Category[O1, M1], d: Category[O2, M2]) {
    product_category(c, d).identity = product_identity(c, d)
} by {
    let p = product_category(c, d)
    (Category[Pair[O1, O2], Pair[M1, M2]].new(product_src(c, d), product_dst(c, d), product_identity(c, d), product_compose(c, d)) = Option.some(p))
    category_new_identity(product_src(c, d), product_dst(c, d), product_identity(c, d), product_compose(c, d), p)
}

/// The composition operator of the product category is `product_compose`.
theorem product_category_compose[O1, M1, O2, M2](c: Category[O1, M1], d: Category[O2, M2]) {
    product_category(c, d).compose = product_compose(c, d)
} by {
    let p = product_category(c, d)
    (Category[Pair[O1, O2], Pair[M1, M2]].new(product_src(c, d), product_dst(c, d), product_identity(c, d), product_compose(c, d)) = Option.some(p))
    category_new_compose(product_src(c, d), product_dst(c, d), product_identity(c, d), product_compose(c, d), p)
}
