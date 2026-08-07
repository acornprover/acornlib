/// Functors preserve categorical isomorphisms.

from category.functor import Functor, functor_src, functor_dst, functor_identity, functor_compose
from category.category_iso import Iso, is_iso_pair, iso_is_iso_pair, iso_new_cat, iso_new_hom,
    iso_new_inv

/// A functor preserves the source-target equation of the forward image.
theorem functor_preserves_iso_pair_src_hom[O1, M1, O2, M2](
    ftr: Functor[O1, M1, O2, M2], f: M1, g: M1) {
    is_iso_pair(ftr.src_cat, f, g) implies
        ftr.dst_cat.src(ftr.mor_map(f)) = ftr.dst_cat.dst(ftr.mor_map(g))
} by {
    if is_iso_pair(ftr.src_cat, f, g) {
        let c = ftr.src_cat
        let d = ftr.dst_cat
        let ff = ftr.mor_map(f)
        let gg = ftr.mor_map(g)
        is_iso_pair(c, f, g) = (c.src(f) = c.dst(g)
            and c.src(g) = c.dst(f)
            and c.compose(g, f) = c.identity(c.src(f))
            and c.compose(f, g) = c.identity(c.dst(f)))
        functor_src(ftr, f)
        functor_dst(ftr, g)
        ftr.obj_map(c.src(f)) = ftr.obj_map(c.dst(g))
        ftr.dst_cat.src(ftr.mor_map(f)) = ftr.dst_cat.dst(ftr.mor_map(g))
    }
}

/// A functor preserves the source-target equation of the inverse image.
theorem functor_preserves_iso_pair_src_inv[O1, M1, O2, M2](
    ftr: Functor[O1, M1, O2, M2], f: M1, g: M1) {
    is_iso_pair(ftr.src_cat, f, g) implies
        ftr.dst_cat.src(ftr.mor_map(g)) = ftr.dst_cat.dst(ftr.mor_map(f))
} by {
    if is_iso_pair(ftr.src_cat, f, g) {
        let c = ftr.src_cat
        let d = ftr.dst_cat
        let ff = ftr.mor_map(f)
        let gg = ftr.mor_map(g)
        is_iso_pair(c, f, g) = (c.src(f) = c.dst(g)
            and c.src(g) = c.dst(f)
            and c.compose(g, f) = c.identity(c.src(f))
            and c.compose(f, g) = c.identity(c.dst(f)))
        functor_src(ftr, g)
        functor_dst(ftr, f)
        ftr.obj_map(c.src(g)) = ftr.obj_map(c.dst(f))
        ftr.dst_cat.src(ftr.mor_map(g)) = ftr.dst_cat.dst(ftr.mor_map(f))
    }
}

/// A functor preserves the inverse-after-forward identity equation.
theorem functor_preserves_iso_pair_inv_hom[O1, M1, O2, M2](
    ftr: Functor[O1, M1, O2, M2], f: M1, g: M1) {
    is_iso_pair(ftr.src_cat, f, g) implies
        ftr.dst_cat.compose(ftr.mor_map(g), ftr.mor_map(f)) =
            ftr.dst_cat.identity(ftr.dst_cat.src(ftr.mor_map(f)))
} by {
    if is_iso_pair(ftr.src_cat, f, g) {
        let c = ftr.src_cat
        let d = ftr.dst_cat
        let ff = ftr.mor_map(f)
        let gg = ftr.mor_map(g)
        is_iso_pair(c, f, g) = (c.src(f) = c.dst(g)
            and c.src(g) = c.dst(f)
            and c.compose(g, f) = c.identity(c.src(f))
            and c.compose(f, g) = c.identity(c.dst(f)))
        c.src(g) = c.dst(f)
        functor_compose(ftr, g, f)
        ftr.mor_map(ftr.src_cat.compose(g, f)) =
            ftr.dst_cat.compose(ftr.mor_map(g), ftr.mor_map(f))
        ftr.mor_map(c.compose(g, f)) = ftr.mor_map(c.identity(c.src(f)))
        functor_identity(ftr, c.src(f))
        functor_src(ftr, f)
        ftr.dst_cat.compose(ftr.mor_map(g), ftr.mor_map(f)) =
            ftr.dst_cat.identity(ftr.dst_cat.src(ftr.mor_map(f)))
    }
}

/// A functor preserves the forward-after-inverse identity equation.
theorem functor_preserves_iso_pair_hom_inv[O1, M1, O2, M2](
    ftr: Functor[O1, M1, O2, M2], f: M1, g: M1) {
    is_iso_pair(ftr.src_cat, f, g) implies
        ftr.dst_cat.compose(ftr.mor_map(f), ftr.mor_map(g)) =
            ftr.dst_cat.identity(ftr.dst_cat.dst(ftr.mor_map(f)))
} by {
    if is_iso_pair(ftr.src_cat, f, g) {
        let c = ftr.src_cat
        let d = ftr.dst_cat
        let ff = ftr.mor_map(f)
        let gg = ftr.mor_map(g)
        is_iso_pair(c, f, g) = (c.src(f) = c.dst(g)
            and c.src(g) = c.dst(f)
            and c.compose(g, f) = c.identity(c.src(f))
            and c.compose(f, g) = c.identity(c.dst(f)))
        c.src(f) = c.dst(g)
        functor_compose(ftr, f, g)
        ftr.mor_map(ftr.src_cat.compose(f, g)) =
            ftr.dst_cat.compose(ftr.mor_map(f), ftr.mor_map(g))
        ftr.mor_map(c.compose(f, g)) = ftr.mor_map(c.identity(c.dst(f)))
        functor_identity(ftr, c.dst(f))
        functor_dst(ftr, f)
        ftr.dst_cat.compose(ftr.mor_map(f), ftr.mor_map(g)) =
            ftr.dst_cat.identity(ftr.dst_cat.dst(ftr.mor_map(f)))
    }
}

/// A functor sends an inverse pair in its source category to an inverse pair in its target category.
theorem functor_preserves_iso_pair[O1, M1, O2, M2](
    ftr: Functor[O1, M1, O2, M2], f: M1, g: M1) {
    is_iso_pair(ftr.src_cat, f, g) implies
        is_iso_pair(ftr.dst_cat, ftr.mor_map(f), ftr.mor_map(g))
} by {
    if is_iso_pair(ftr.src_cat, f, g) {
        let d = ftr.dst_cat
        let ff = ftr.mor_map(f)
        let gg = ftr.mor_map(g)
        functor_preserves_iso_pair_src_hom(ftr, f, g)
        functor_preserves_iso_pair_src_inv(ftr, f, g)
        d.src(gg) = d.dst(ff)
        functor_preserves_iso_pair_inv_hom(ftr, f, g)
        d.compose(gg, ff) = d.identity(d.src(ff))
        functor_preserves_iso_pair_hom_inv(ftr, f, g)
        d.compose(ff, gg) = d.identity(d.dst(ff))
        is_iso_pair(ftr.dst_cat, ftr.mor_map(f), ftr.mor_map(g))
    }
}

/// A functor maps a bundled isomorphism over its source category to a bundled isomorphism over its target category.
theorem functor_map_iso_exists[O1, M1, O2, M2](
    ftr: Functor[O1, M1, O2, M2], e: Iso[O1, M1]) {
    e.cat = ftr.src_cat implies exists(r: Iso[O2, M2]) {
        r.cat = ftr.dst_cat and r.hom = ftr.mor_map(e.hom) and r.inv = ftr.mor_map(e.inv)
    }
} by {
    if e.cat = ftr.src_cat {
        iso_is_iso_pair(e)
        functor_preserves_iso_pair(ftr, e.hom, e.inv)
        is_iso_pair(ftr.dst_cat, ftr.mor_map(e.hom), ftr.mor_map(e.inv))
        let r: Iso[O2, M2] satisfy {
            Iso[O2, M2].new(ftr.dst_cat, ftr.mor_map(e.hom), ftr.mor_map(e.inv)) = Option.some(r)
        }
        iso_new_cat(ftr.dst_cat, ftr.mor_map(e.hom), ftr.mor_map(e.inv), r)
        iso_new_hom(ftr.dst_cat, ftr.mor_map(e.hom), ftr.mor_map(e.inv), r)
        iso_new_inv(ftr.dst_cat, ftr.mor_map(e.hom), ftr.mor_map(e.inv), r)
        exists(h: Iso[O2, M2]) {
            h.cat = ftr.dst_cat and h.hom = ftr.mor_map(e.hom) and h.inv = ftr.mor_map(e.inv)
        }
    }
}
