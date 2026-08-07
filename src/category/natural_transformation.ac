/// Natural transformations between functors.

from category.category import Category, category_identity_src, category_identity_dst,
    category_id_left, category_id_right, category_compose_src, category_compose_dst,
    category_compose_assoc
from category.functor import Functor, functor_src, functor_dst, functor_identity, functor_compose,
    compose_functor, compose_functor_obj_map, compose_functor_mor_map,
    compose_functor_src_cat, compose_functor_dst_cat
from data.basic.functions import compose

/// True if the two functors share the same source and target categories.
define functors_parallel[O1, M1, O2, M2](
    f: Functor[O1, M1, O2, M2], g: Functor[O1, M1, O2, M2]
) -> Bool {
    f.src_cat = g.src_cat and f.dst_cat = g.dst_cat
}

/// True if `component(x)` is a morphism from `f.obj_map(x)` to `g.obj_map(x)` in the target category.
define nat_trans_endpoints_axiom[O1, M1, O2, M2](
    f: Functor[O1, M1, O2, M2], g: Functor[O1, M1, O2, M2], component: O1 -> M2
) -> Bool {
    forall(x: O1) {
        f.dst_cat.src(component(x)) = f.obj_map(x)
        and f.dst_cat.dst(component(x)) = g.obj_map(x)
    }
}

/// True if the naturality square commutes for every morphism in the source category.
define nat_trans_naturality_axiom[O1, M1, O2, M2](
    f: Functor[O1, M1, O2, M2], g: Functor[O1, M1, O2, M2], component: O1 -> M2
) -> Bool {
    forall(m: M1) {
        f.dst_cat.compose(component(f.src_cat.dst(m)), f.mor_map(m))
            = f.dst_cat.compose(g.mor_map(m), component(f.src_cat.src(m)))
    }
}

/// True if `component` defines a natural transformation from `f` to `g`.
define is_natural_transformation[O1, M1, O2, M2](
    f: Functor[O1, M1, O2, M2], g: Functor[O1, M1, O2, M2], component: O1 -> M2
) -> Bool {
    functors_parallel(f, g)
    and nat_trans_endpoints_axiom(f, g, component)
    and nat_trans_naturality_axiom(f, g, component)
}

/// A natural transformation between two functors with matching source and target categories.
structure NaturalTransformation[O1, M1, O2, M2] {
    /// The source functor.
    src_functor: Functor[O1, M1, O2, M2]

    /// The target functor.
    dst_functor: Functor[O1, M1, O2, M2]

    /// The component morphism at each object of the source category.
    component: O1 -> M2
} constraint {
    is_natural_transformation(src_functor, dst_functor, component)
}

/// Construction of a natural transformation remembers its source functor.
theorem nat_trans_new_src_functor[O1, M1, O2, M2](
    f: Functor[O1, M1, O2, M2], g: Functor[O1, M1, O2, M2], component: O1 -> M2,
    n: NaturalTransformation[O1, M1, O2, M2]
) {
    NaturalTransformation[O1, M1, O2, M2].new(f, g, component) = Option.some(n)
        implies n.src_functor = f
}

/// Construction of a natural transformation remembers its target functor.
theorem nat_trans_new_dst_functor[O1, M1, O2, M2](
    f: Functor[O1, M1, O2, M2], g: Functor[O1, M1, O2, M2], component: O1 -> M2,
    n: NaturalTransformation[O1, M1, O2, M2]
) {
    NaturalTransformation[O1, M1, O2, M2].new(f, g, component) = Option.some(n)
        implies n.dst_functor = g
}

/// Construction of a natural transformation remembers its component map.
theorem nat_trans_new_component[O1, M1, O2, M2](
    f: Functor[O1, M1, O2, M2], g: Functor[O1, M1, O2, M2], component: O1 -> M2,
    n: NaturalTransformation[O1, M1, O2, M2]
) {
    NaturalTransformation[O1, M1, O2, M2].new(f, g, component) = Option.some(n)
        implies n.component = component
}

/// A natural transformation's data is a natural transformation.
theorem nat_trans_is_natural_transformation[O1, M1, O2, M2](
    n: NaturalTransformation[O1, M1, O2, M2]
) {
    is_natural_transformation(n.src_functor, n.dst_functor, n.component)
} by {
}

/// The source and target functors of a natural transformation share a source category.
theorem nat_trans_shared_src_cat[O1, M1, O2, M2](n: NaturalTransformation[O1, M1, O2, M2]) {
    n.src_functor.src_cat = n.dst_functor.src_cat
} by {
    nat_trans_is_natural_transformation(n)
    is_natural_transformation(n.src_functor, n.dst_functor, n.component) =
        (functors_parallel(n.src_functor, n.dst_functor)
        and nat_trans_endpoints_axiom(n.src_functor, n.dst_functor, n.component)
        and nat_trans_naturality_axiom(n.src_functor, n.dst_functor, n.component))
}

/// The source and target functors of a natural transformation share a target category.
theorem nat_trans_shared_dst_cat[O1, M1, O2, M2](n: NaturalTransformation[O1, M1, O2, M2]) {
    n.src_functor.dst_cat = n.dst_functor.dst_cat
} by {
    nat_trans_is_natural_transformation(n)
    is_natural_transformation(n.src_functor, n.dst_functor, n.component) =
        (functors_parallel(n.src_functor, n.dst_functor)
        and nat_trans_endpoints_axiom(n.src_functor, n.dst_functor, n.component)
        and nat_trans_naturality_axiom(n.src_functor, n.dst_functor, n.component))
}

/// The component morphism at any object has the source functor's image as its source.
theorem nat_trans_component_src[O1, M1, O2, M2](
    n: NaturalTransformation[O1, M1, O2, M2], x: O1
) {
    n.src_functor.dst_cat.src(n.component(x)) = n.src_functor.obj_map(x)
} by {
    nat_trans_is_natural_transformation(n)
    is_natural_transformation(n.src_functor, n.dst_functor, n.component) =
        (functors_parallel(n.src_functor, n.dst_functor)
        and nat_trans_endpoints_axiom(n.src_functor, n.dst_functor, n.component)
        and nat_trans_naturality_axiom(n.src_functor, n.dst_functor, n.component))
    nat_trans_endpoints_axiom(n.src_functor, n.dst_functor, n.component) = forall(y: O1) {
        n.src_functor.dst_cat.src(n.component(y)) = n.src_functor.obj_map(y)
        and n.src_functor.dst_cat.dst(n.component(y)) = n.dst_functor.obj_map(y)
    }
}

/// The component morphism at any object has the target functor's image as its target.
theorem nat_trans_component_dst[O1, M1, O2, M2](
    n: NaturalTransformation[O1, M1, O2, M2], x: O1
) {
    n.src_functor.dst_cat.dst(n.component(x)) = n.dst_functor.obj_map(x)
} by {
    nat_trans_is_natural_transformation(n)
    is_natural_transformation(n.src_functor, n.dst_functor, n.component) =
        (functors_parallel(n.src_functor, n.dst_functor)
        and nat_trans_endpoints_axiom(n.src_functor, n.dst_functor, n.component)
        and nat_trans_naturality_axiom(n.src_functor, n.dst_functor, n.component))
    nat_trans_endpoints_axiom(n.src_functor, n.dst_functor, n.component) = forall(y: O1) {
        n.src_functor.dst_cat.src(n.component(y)) = n.src_functor.obj_map(y)
        and n.src_functor.dst_cat.dst(n.component(y)) = n.dst_functor.obj_map(y)
    }
}

/// The naturality square commutes for any morphism in the source category.
theorem nat_trans_naturality[O1, M1, O2, M2](
    n: NaturalTransformation[O1, M1, O2, M2], m: M1
) {
    n.src_functor.dst_cat.compose(n.component(n.src_functor.src_cat.dst(m)), n.src_functor.mor_map(m))
        = n.src_functor.dst_cat.compose(n.dst_functor.mor_map(m), n.component(n.src_functor.src_cat.src(m)))
} by {
    nat_trans_is_natural_transformation(n)
    is_natural_transformation(n.src_functor, n.dst_functor, n.component) =
        (functors_parallel(n.src_functor, n.dst_functor)
        and nat_trans_endpoints_axiom(n.src_functor, n.dst_functor, n.component)
        and nat_trans_naturality_axiom(n.src_functor, n.dst_functor, n.component))
    nat_trans_naturality_axiom(n.src_functor, n.dst_functor, n.component)
}


/// The component morphism of the identity natural transformation on a functor.
define identity_nat_trans_component[O1, M1, O2, M2](f: Functor[O1, M1, O2, M2], x: O1) -> M2 {
    f.dst_cat.identity(f.obj_map(x))
}

/// The identity component map satisfies the endpoints axiom.
theorem identity_nat_trans_endpoints[O1, M1, O2, M2](f: Functor[O1, M1, O2, M2]) {
    nat_trans_endpoints_axiom(f, f, identity_nat_trans_component(f))
} by {
    forall(x: O1) {
        identity_nat_trans_component(f)(x) = identity_nat_trans_component(f, x)
        category_identity_src(f.dst_cat, f.obj_map(x))
        category_identity_dst(f.dst_cat, f.obj_map(x))
        f.dst_cat.dst(identity_nat_trans_component(f)(x)) = f.obj_map(x)
    }
}

/// For the identity natural transformation, the left side of the naturality square reduces to the morphism itself.
theorem identity_nat_trans_naturality_left[O1, M1, O2, M2](f: Functor[O1, M1, O2, M2], m: M1) {
    f.dst_cat.compose(identity_nat_trans_component(f)(f.src_cat.dst(m)), f.mor_map(m)) = f.mor_map(m)
} by {
    identity_nat_trans_component(f)(f.src_cat.dst(m)) = identity_nat_trans_component(f, f.src_cat.dst(m))
    functor_dst(f, m)
    category_id_left(f.dst_cat, f.mor_map(m))
}

/// For the identity natural transformation, the right side of the naturality square reduces to the morphism itself.
theorem identity_nat_trans_naturality_right[O1, M1, O2, M2](f: Functor[O1, M1, O2, M2], m: M1) {
    f.dst_cat.compose(f.mor_map(m), identity_nat_trans_component(f)(f.src_cat.src(m))) = f.mor_map(m)
} by {
    identity_nat_trans_component(f)(f.src_cat.src(m)) = identity_nat_trans_component(f, f.src_cat.src(m))
    functor_src(f, m)
    category_id_right(f.dst_cat, f.mor_map(m))
}

/// The identity component map satisfies the naturality axiom.
theorem identity_nat_trans_naturality[O1, M1, O2, M2](f: Functor[O1, M1, O2, M2]) {
    nat_trans_naturality_axiom(f, f, identity_nat_trans_component(f))
} by {
    forall(m: M1) {
        identity_nat_trans_naturality_left(f, m)
        identity_nat_trans_naturality_right(f, m)
    }
}

/// The identity component map is a natural transformation from a functor to itself.
theorem identity_is_nat_trans[O1, M1, O2, M2](f: Functor[O1, M1, O2, M2]) {
    is_natural_transformation(f, f, identity_nat_trans_component(f))
} by {
    functors_parallel(f, f)
    identity_nat_trans_endpoints(f)
    identity_nat_trans_naturality(f)
}

/// The identity natural transformation on a functor.
let identity_nat_trans[O1, M1, O2, M2](f: Functor[O1, M1, O2, M2]) -> result: NaturalTransformation[O1, M1, O2, M2] satisfy {
    NaturalTransformation[O1, M1, O2, M2].new(f, f, identity_nat_trans_component(f)) = Option.some(result)
} by {
    identity_is_nat_trans(f)
    let n: NaturalTransformation[O1, M1, O2, M2] satisfy {
        NaturalTransformation[O1, M1, O2, M2].new(f, f, identity_nat_trans_component(f)) = Option.some(n)
    }
}

/// The identity natural transformation has the original functor as its source functor.
theorem identity_nat_trans_src_functor[O1, M1, O2, M2](f: Functor[O1, M1, O2, M2]) {
    identity_nat_trans(f).src_functor = f
} by {
    let n = identity_nat_trans(f)
    NaturalTransformation[O1, M1, O2, M2].new(f, f, identity_nat_trans_component(f)) = Option.some(n)
    nat_trans_new_src_functor(f, f, identity_nat_trans_component(f), n)
}

/// The identity natural transformation has the original functor as its target functor.
theorem identity_nat_trans_dst_functor[O1, M1, O2, M2](f: Functor[O1, M1, O2, M2]) {
    identity_nat_trans(f).dst_functor = f
} by {
    let n = identity_nat_trans(f)
    NaturalTransformation[O1, M1, O2, M2].new(f, f, identity_nat_trans_component(f)) = Option.some(n)
    nat_trans_new_dst_functor(f, f, identity_nat_trans_component(f), n)
}

/// The identity natural transformation has the per-object identity as its component map.
theorem identity_nat_trans_component_eq[O1, M1, O2, M2](f: Functor[O1, M1, O2, M2]) {
    identity_nat_trans(f).component = identity_nat_trans_component(f)
} by {
    let n = identity_nat_trans(f)
    NaturalTransformation[O1, M1, O2, M2].new(f, f, identity_nat_trans_component(f)) = Option.some(n)
    nat_trans_new_component(f, f, identity_nat_trans_component(f), n)
}

/// True if two natural transformations can be composed vertically: the inner natural transformation's target functor matches the outer's source functor.
define vertical_composable[O1, M1, O2, M2](
    upper: NaturalTransformation[O1, M1, O2, M2],
    lower: NaturalTransformation[O1, M1, O2, M2]
) -> Bool {
    upper.src_functor = lower.dst_functor
}

/// The component morphism of the vertical composition of two natural transformations at an object.
define vertical_compose_component[O1, M1, O2, M2](
    upper: NaturalTransformation[O1, M1, O2, M2],
    lower: NaturalTransformation[O1, M1, O2, M2],
    x: O1
) -> M2 {
    upper.src_functor.dst_cat.compose(upper.component(x), lower.component(x))
}

/// Vertically composable natural transformations share the same target category.
theorem vertical_composable_dst_cat[O1, M1, O2, M2](
    upper: NaturalTransformation[O1, M1, O2, M2],
    lower: NaturalTransformation[O1, M1, O2, M2]
) {
    vertical_composable(upper, lower) implies
        upper.src_functor.dst_cat = lower.src_functor.dst_cat
        and upper.dst_functor.dst_cat = lower.src_functor.dst_cat
} by {
    if vertical_composable(upper, lower) {
        upper.src_functor = lower.dst_functor
        nat_trans_shared_dst_cat(lower)
        nat_trans_shared_dst_cat(upper)
        upper.dst_functor.dst_cat = lower.src_functor.dst_cat
    }
}

/// Vertically composable natural transformations share the same source category.
theorem vertical_composable_src_cat[O1, M1, O2, M2](
    upper: NaturalTransformation[O1, M1, O2, M2],
    lower: NaturalTransformation[O1, M1, O2, M2]
) {
    vertical_composable(upper, lower) implies
        upper.src_functor.src_cat = lower.src_functor.src_cat
        and upper.dst_functor.src_cat = lower.src_functor.src_cat
} by {
    if vertical_composable(upper, lower) {
        upper.src_functor = lower.dst_functor
        nat_trans_shared_src_cat(lower)
        nat_trans_shared_src_cat(upper)
        upper.dst_functor.src_cat = lower.src_functor.src_cat
    }
}


/// Under composability, the upper natural transformation's source functor obj_map agrees with the lower's target functor obj_map.
theorem vertical_compose_obj_map_eq[O1, M1, O2, M2](
    upper: NaturalTransformation[O1, M1, O2, M2],
    lower: NaturalTransformation[O1, M1, O2, M2],
    x: O1
) {
    vertical_composable(upper, lower) implies
        upper.src_functor.obj_map(x) = lower.dst_functor.obj_map(x)
} by {
    if vertical_composable(upper, lower) {
        upper.src_functor = lower.dst_functor
    }
}

/// Under composability, lower's target functor dst_cat equals upper's source functor dst_cat applied via field congruence.
theorem vertical_compose_dst_cat_lower_dst_eq_upper_src[O1, M1, O2, M2](
    upper: NaturalTransformation[O1, M1, O2, M2],
    lower: NaturalTransformation[O1, M1, O2, M2]
) {
    vertical_composable(upper, lower) implies
        lower.dst_functor.dst_cat = upper.src_functor.dst_cat
} by {
    if vertical_composable(upper, lower) {
        upper.src_functor = lower.dst_functor
    }
}

/// Targeted: in lower's frame, the dst of lower's component equals lower's dst_functor.obj_map.
theorem vc_lower_component_dst[O1, M1, O2, M2](
    lower: NaturalTransformation[O1, M1, O2, M2], x: O1
) {
    lower.src_functor.dst_cat.dst(lower.component(x)) = lower.dst_functor.obj_map(x)
} by {
    nat_trans_component_dst(lower, x)
}

/// Under composability, lower's source functor target category equals upper's source functor target category.
theorem vc_dst_cat_eq[O1, M1, O2, M2](
    upper: NaturalTransformation[O1, M1, O2, M2],
    lower: NaturalTransformation[O1, M1, O2, M2]
) {
    vertical_composable(upper, lower) implies
        lower.src_functor.dst_cat = upper.src_functor.dst_cat
} by {
    if vertical_composable(upper, lower) {
        nat_trans_shared_dst_cat(lower)
        vertical_compose_dst_cat_lower_dst_eq_upper_src(upper, lower)
        lower.dst_functor.dst_cat = upper.src_functor.dst_cat
    }
}

/// Substitute one category for another in dst applied to a morphism.
theorem vc_dst_substitute[O, M](c: Category[O, M], d: Category[O, M], m: M) {
    c = d implies c.dst(m) = d.dst(m)
}

/// Substitute one category for another in src applied to a morphism.
theorem vc_src_substitute[O, M](c: Category[O, M], d: Category[O, M], m: M) {
    c = d implies c.src(m) = d.src(m)
}

/// Generic: in equal categories, the same morphism has the same target.
theorem cat_eq_dst_eq[O, M](c: Category[O, M], d: Category[O, M], m: M, y: O) {
    c = d and c.dst(m) = y implies d.dst(m) = y
}

/// Under composability, in upper's dst_cat, the dst of lower's component equals lower's dst_functor obj_map.
theorem vc_lower_dst_upper_cat[O1, M1, O2, M2](
    upper: NaturalTransformation[O1, M1, O2, M2],
    lower: NaturalTransformation[O1, M1, O2, M2],
    x: O1
) {
    vertical_composable(upper, lower) implies
        upper.src_functor.dst_cat.dst(lower.component(x)) = lower.dst_functor.obj_map(x)
} by {
    if vertical_composable(upper, lower) {
        vc_dst_cat_eq(upper, lower)
        vc_lower_component_dst(lower, x)
        cat_eq_dst_eq(lower.src_functor.dst_cat, upper.src_functor.dst_cat,
            lower.component(x), lower.dst_functor.obj_map(x))
    }
}

/// Generic transitive helper.
theorem vc_eq_chain[T](a: T, b: T, c: T) {
    a = b and b = c implies a = c
}

/// Under composability, the upper natural transformation's source functor obj_map equals the lower's target functor obj_map at any object (reversed direction of vertical_compose_obj_map_eq).
theorem vc_lower_dst_obj_map_eq_upper_src[O1, M1, O2, M2](
    upper: NaturalTransformation[O1, M1, O2, M2],
    lower: NaturalTransformation[O1, M1, O2, M2],
    x: O1
) {
    vertical_composable(upper, lower) implies
        lower.dst_functor.obj_map(x) = upper.src_functor.obj_map(x)
} by {
    if vertical_composable(upper, lower) {
        upper.src_functor = lower.dst_functor
    }
}

/// Under composability, in upper's dst_cat, the dst of lower's component equals upper's obj_map.
theorem vc_lower_component_dst_in_upper_cat[O1, M1, O2, M2](
    upper: NaturalTransformation[O1, M1, O2, M2],
    lower: NaturalTransformation[O1, M1, O2, M2],
    x: O1
) {
    vertical_composable(upper, lower) implies
        upper.src_functor.dst_cat.dst(lower.component(x)) = upper.src_functor.obj_map(x)
} by {
    if vertical_composable(upper, lower) {
        upper.src_functor.obj_map(x) = lower.dst_functor.obj_map(x)
        vc_lower_dst_upper_cat(upper, lower, x)
        upper.src_functor.dst_cat.dst(lower.component(x)) = upper.src_functor.obj_map(x)
    }
}

/// At each object, the component pair of a vertically composable transformation pair is composable in upper's target category.
theorem vertical_compose_components_composable[O1, M1, O2, M2](
    upper: NaturalTransformation[O1, M1, O2, M2],
    lower: NaturalTransformation[O1, M1, O2, M2],
    x: O1
) {
    vertical_composable(upper, lower) implies
        upper.src_functor.dst_cat.src(upper.component(x)) = upper.src_functor.dst_cat.dst(lower.component(x))
} by {
    if vertical_composable(upper, lower) {
        nat_trans_component_src(upper, x)
        vc_lower_component_dst_in_upper_cat(upper, lower, x)
        upper.src_functor.dst_cat.dst(lower.component(x)) = upper.src_functor.obj_map(x)
    }
}

/// Generic substitution: equal categories give equal src applied to a morphism, with chained equality.
theorem cat_eq_src_eq[O, M](c: Category[O, M], d: Category[O, M], m: M, y: O) {
    c = d and c.src(m) = y implies d.src(m) = y
}

/// Under composability, the src of the lower component in upper's dst_cat is lower's source functor obj_map.
theorem vc_lower_component_src_in_upper_cat[O1, M1, O2, M2](
    upper: NaturalTransformation[O1, M1, O2, M2],
    lower: NaturalTransformation[O1, M1, O2, M2],
    x: O1
) {
    vertical_composable(upper, lower) implies
        upper.src_functor.dst_cat.src(lower.component(x)) = lower.src_functor.obj_map(x)
} by {
    if vertical_composable(upper, lower) {
        vc_dst_cat_eq(upper, lower)
        nat_trans_component_src(lower, x)
        cat_eq_src_eq(lower.src_functor.dst_cat, upper.src_functor.dst_cat,
            lower.component(x), lower.src_functor.obj_map(x))
    }
}

/// Step: Under composability, the components of upper and lower are composable in upper's dst_cat.
theorem vc_compose_src_step1[O1, M1, O2, M2](
    upper: NaturalTransformation[O1, M1, O2, M2],
    lower: NaturalTransformation[O1, M1, O2, M2],
    x: O1
) {
    vertical_composable(upper, lower) implies
        upper.src_functor.dst_cat.src(upper.src_functor.dst_cat.compose(upper.component(x), lower.component(x)))
            = upper.src_functor.dst_cat.src(lower.component(x))
} by {
    if vertical_composable(upper, lower) {
        vertical_compose_components_composable(upper, lower, x)
        category_compose_src(upper.src_functor.dst_cat, upper.component(x), lower.component(x))
    }
}

/// Under composability, the src of the composition of upper and lower components in upper's dst_cat is lower's source functor obj_map.
theorem vc_compose_src_eq[O1, M1, O2, M2](
    upper: NaturalTransformation[O1, M1, O2, M2],
    lower: NaturalTransformation[O1, M1, O2, M2],
    x: O1
) {
    vertical_composable(upper, lower) implies
        upper.src_functor.dst_cat.src(upper.src_functor.dst_cat.compose(upper.component(x), lower.component(x))) = lower.src_functor.obj_map(x)
} by {
    if vertical_composable(upper, lower) {
        vc_compose_src_step1(upper, lower, x)
        vc_lower_component_src_in_upper_cat(upper, lower, x)
        upper.src_functor.dst_cat.src(upper.src_functor.dst_cat.compose(upper.component(x), lower.component(x))) = lower.src_functor.obj_map(x)
    }
}

/// Under composability, the src of the vertical composition's component is the source functor's image at that object.
theorem vc_endpoint_src[O1, M1, O2, M2](
    upper: NaturalTransformation[O1, M1, O2, M2],
    lower: NaturalTransformation[O1, M1, O2, M2],
    x: O1
) {
    vertical_composable(upper, lower) implies
        upper.src_functor.dst_cat.src(vertical_compose_component(upper, lower, x)) = lower.src_functor.obj_map(x)
} by {
    if vertical_composable(upper, lower) {
        vertical_compose_component(upper, lower, x) = upper.src_functor.dst_cat.compose(upper.component(x), lower.component(x))
        vc_compose_src_eq(upper, lower, x)
    }
}

/// Step: Under composability, the components of upper and lower are composable in upper's dst_cat (target side).
theorem vc_compose_dst_step1[O1, M1, O2, M2](
    upper: NaturalTransformation[O1, M1, O2, M2],
    lower: NaturalTransformation[O1, M1, O2, M2],
    x: O1
) {
    vertical_composable(upper, lower) implies
        upper.src_functor.dst_cat.dst(upper.src_functor.dst_cat.compose(upper.component(x), lower.component(x)))
            = upper.src_functor.dst_cat.dst(upper.component(x))
} by {
    if vertical_composable(upper, lower) {
        vertical_compose_components_composable(upper, lower, x)
        category_compose_dst(upper.src_functor.dst_cat, upper.component(x), lower.component(x))
    }
}

/// Under composability, the dst of upper's component in upper's dst_cat is upper's target functor obj_map.
theorem vc_upper_component_dst_in_upper_cat[O1, M1, O2, M2](
    upper: NaturalTransformation[O1, M1, O2, M2], x: O1
) {
    upper.src_functor.dst_cat.dst(upper.component(x)) = upper.dst_functor.obj_map(x)
} by {
    nat_trans_component_dst(upper, x)
}

/// Under composability, the dst of the composition of upper and lower components in upper's dst_cat is upper's target functor obj_map.
theorem vc_compose_dst_eq[O1, M1, O2, M2](
    upper: NaturalTransformation[O1, M1, O2, M2],
    lower: NaturalTransformation[O1, M1, O2, M2],
    x: O1
) {
    vertical_composable(upper, lower) implies
        upper.src_functor.dst_cat.dst(upper.src_functor.dst_cat.compose(upper.component(x), lower.component(x))) = upper.dst_functor.obj_map(x)
} by {
    if vertical_composable(upper, lower) {
        vc_compose_dst_step1(upper, lower, x)
        vc_upper_component_dst_in_upper_cat(upper, x)
        upper.src_functor.dst_cat.dst(upper.src_functor.dst_cat.compose(upper.component(x), lower.component(x))) = upper.dst_functor.obj_map(x)
    }
}

/// Under composability, the dst of the vertical composition's component is the target functor's image at that object.
theorem vc_endpoint_dst[O1, M1, O2, M2](
    upper: NaturalTransformation[O1, M1, O2, M2],
    lower: NaturalTransformation[O1, M1, O2, M2],
    x: O1
) {
    vertical_composable(upper, lower) implies
        upper.src_functor.dst_cat.dst(vertical_compose_component(upper, lower, x)) = upper.dst_functor.obj_map(x)
} by {
    if vertical_composable(upper, lower) {
        vertical_compose_component(upper, lower, x) = upper.src_functor.dst_cat.compose(upper.component(x), lower.component(x))
        vc_compose_dst_eq(upper, lower, x)
    }
}

/// Under composability, the src/dst endpoints of the vertical composition's component lie in lower's source dst_cat.
theorem vc_endpoint_src_lower_cat[O1, M1, O2, M2](
    upper: NaturalTransformation[O1, M1, O2, M2],
    lower: NaturalTransformation[O1, M1, O2, M2],
    x: O1
) {
    vertical_composable(upper, lower) implies
        lower.src_functor.dst_cat.src(vertical_compose_component(upper, lower, x)) = lower.src_functor.obj_map(x)
} by {
    if vertical_composable(upper, lower) {
        vc_endpoint_src(upper, lower, x)
        vc_dst_cat_eq(upper, lower)
        lower.src_functor.dst_cat.src(vertical_compose_component(upper, lower, x)) = lower.src_functor.obj_map(x)
    }
}

/// Under composability, the dst endpoint of the vertical composition's component lies in lower's source dst_cat.
theorem vc_endpoint_dst_lower_cat[O1, M1, O2, M2](
    upper: NaturalTransformation[O1, M1, O2, M2],
    lower: NaturalTransformation[O1, M1, O2, M2],
    x: O1
) {
    vertical_composable(upper, lower) implies
        lower.src_functor.dst_cat.dst(vertical_compose_component(upper, lower, x)) = upper.dst_functor.obj_map(x)
} by {
    if vertical_composable(upper, lower) {
        vc_endpoint_dst(upper, lower, x)
        vc_dst_cat_eq(upper, lower)
        lower.src_functor.dst_cat.dst(vertical_compose_component(upper, lower, x)) = upper.dst_functor.obj_map(x)
    }
}

/// Under composability, the vertical compose component map satisfies the endpoints axiom.
theorem vc_endpoints_axiom[O1, M1, O2, M2](
    upper: NaturalTransformation[O1, M1, O2, M2],
    lower: NaturalTransformation[O1, M1, O2, M2]
) {
    vertical_composable(upper, lower) implies
        nat_trans_endpoints_axiom(lower.src_functor, upper.dst_functor, vertical_compose_component(upper, lower))
} by {
    if vertical_composable(upper, lower) {
        forall(x: O1) {
            vc_endpoint_src_lower_cat(upper, lower, x)
            vc_endpoint_dst_lower_cat(upper, lower, x)
            vertical_compose_component(upper, lower)(x) = vertical_compose_component(upper, lower, x)
            lower.src_functor.dst_cat.dst(vertical_compose_component(upper, lower)(x)) = upper.dst_functor.obj_map(x)
        }
    }
}

/// Under composability of nat trans, lower's mor_map src in upper's dst_cat is the source functor's image.
theorem vc_lower_mor_map_src[O1, M1, O2, M2](
    upper: NaturalTransformation[O1, M1, O2, M2],
    lower: NaturalTransformation[O1, M1, O2, M2],
    m: M1
) {
    vertical_composable(upper, lower) implies
        upper.src_functor.dst_cat.src(lower.src_functor.mor_map(m))
            = lower.src_functor.obj_map(lower.src_functor.src_cat.src(m))
} by {
    if vertical_composable(upper, lower) {
        functor_src(lower.src_functor, m)
        vc_dst_cat_eq(upper, lower)
        upper.src_functor.dst_cat.src(lower.src_functor.mor_map(m)) = lower.src_functor.obj_map(lower.src_functor.src_cat.src(m))
    }
}

/// Under composability of nat trans, lower's mor_map dst in upper's dst_cat is the source functor's image.
theorem vc_lower_mor_map_dst[O1, M1, O2, M2](
    upper: NaturalTransformation[O1, M1, O2, M2],
    lower: NaturalTransformation[O1, M1, O2, M2],
    m: M1
) {
    vertical_composable(upper, lower) implies
        upper.src_functor.dst_cat.dst(lower.src_functor.mor_map(m))
            = lower.src_functor.obj_map(lower.src_functor.src_cat.dst(m))
} by {
    if vertical_composable(upper, lower) {
        functor_dst(lower.src_functor, m)
        vc_dst_cat_eq(upper, lower)
        upper.src_functor.dst_cat.dst(lower.src_functor.mor_map(m)) = lower.src_functor.obj_map(lower.src_functor.src_cat.dst(m))
    }
}

/// Under composability, upper's mor_map src in upper's dst_cat is the obj_map at m's source in lower's source category.
theorem vc_upper_mor_map_src[O1, M1, O2, M2](
    upper: NaturalTransformation[O1, M1, O2, M2],
    lower: NaturalTransformation[O1, M1, O2, M2],
    m: M1
) {
    vertical_composable(upper, lower) implies
        upper.src_functor.dst_cat.src(upper.src_functor.mor_map(m))
            = upper.src_functor.obj_map(lower.src_functor.src_cat.src(m))
} by {
    if vertical_composable(upper, lower) {
        functor_src(upper.src_functor, m)
        vertical_composable_src_cat(upper, lower)
        upper.src_functor.dst_cat.src(upper.src_functor.mor_map(m)) = upper.src_functor.obj_map(lower.src_functor.src_cat.src(m))
    }
}

/// Under composability, upper's mor_map dst in upper's dst_cat is the obj_map at m's target in lower's source category.
theorem vc_upper_mor_map_dst[O1, M1, O2, M2](
    upper: NaturalTransformation[O1, M1, O2, M2],
    lower: NaturalTransformation[O1, M1, O2, M2],
    m: M1
) {
    vertical_composable(upper, lower) implies
        upper.src_functor.dst_cat.dst(upper.src_functor.mor_map(m))
            = upper.src_functor.obj_map(lower.src_functor.src_cat.dst(m))
} by {
    if vertical_composable(upper, lower) {
        functor_dst(upper.src_functor, m)
        vertical_composable_src_cat(upper, lower)
        upper.src_functor.dst_cat.dst(upper.src_functor.mor_map(m)) = upper.src_functor.obj_map(lower.src_functor.src_cat.dst(m))
    }
}

/// Under composability, upper's dst_functor mor_map src in upper's dst_cat.
theorem vc_upper_dst_mor_map_src[O1, M1, O2, M2](
    upper: NaturalTransformation[O1, M1, O2, M2],
    lower: NaturalTransformation[O1, M1, O2, M2],
    m: M1
) {
    vertical_composable(upper, lower) implies
        upper.src_functor.dst_cat.src(upper.dst_functor.mor_map(m))
            = upper.dst_functor.obj_map(lower.src_functor.src_cat.src(m))
} by {
    if vertical_composable(upper, lower) {
        functor_src(upper.dst_functor, m)
        nat_trans_shared_dst_cat(upper)
        vertical_composable_src_cat(upper, lower)
        upper.src_functor.dst_cat.src(upper.dst_functor.mor_map(m)) = upper.dst_functor.obj_map(lower.src_functor.src_cat.src(m))
    }
}

/// Lower's naturality, bridged into upper's target category and using upper.src_functor.mor_map for the middle functor.
theorem vc_eta_naturality[O1, M1, O2, M2](
    upper: NaturalTransformation[O1, M1, O2, M2],
    lower: NaturalTransformation[O1, M1, O2, M2],
    m: M1
) {
    vertical_composable(upper, lower) implies
        upper.src_functor.dst_cat.compose(lower.component(lower.src_functor.src_cat.dst(m)), lower.src_functor.mor_map(m))
            = upper.src_functor.dst_cat.compose(upper.src_functor.mor_map(m), lower.component(lower.src_functor.src_cat.src(m)))
} by {
    if vertical_composable(upper, lower) {
        nat_trans_naturality(lower, m)
        vc_dst_cat_eq(upper, lower)
        upper.src_functor.mor_map = lower.dst_functor.mor_map
        upper.src_functor.dst_cat.compose(lower.component(lower.src_functor.src_cat.dst(m)), lower.src_functor.mor_map(m)) = upper.src_functor.dst_cat.compose(upper.src_functor.mor_map(m), lower.component(lower.src_functor.src_cat.src(m)))
    }
}

/// Upper's naturality, bridged so the m-input goes through lower's source category.
theorem vc_theta_naturality[O1, M1, O2, M2](
    upper: NaturalTransformation[O1, M1, O2, M2],
    lower: NaturalTransformation[O1, M1, O2, M2],
    m: M1
) {
    vertical_composable(upper, lower) implies
        upper.src_functor.dst_cat.compose(upper.component(lower.src_functor.src_cat.dst(m)), upper.src_functor.mor_map(m))
            = upper.src_functor.dst_cat.compose(upper.dst_functor.mor_map(m), upper.component(lower.src_functor.src_cat.src(m)))
} by {
    if vertical_composable(upper, lower) {
        nat_trans_naturality(upper, m)
        vertical_composable_src_cat(upper, lower)
        upper.src_functor.dst_cat.compose(upper.component(lower.src_functor.src_cat.dst(m)), upper.src_functor.mor_map(m)) = upper.src_functor.dst_cat.compose(upper.dst_functor.mor_map(m), upper.component(lower.src_functor.src_cat.src(m)))
    }
}

/// Composability: in upper's dst_cat, lower component at y is composable with F_mor(m) where y = C.dst(m).
theorem vc_eta_F_composable[O1, M1, O2, M2](
    upper: NaturalTransformation[O1, M1, O2, M2],
    lower: NaturalTransformation[O1, M1, O2, M2],
    m: M1
) {
    vertical_composable(upper, lower) implies
        upper.src_functor.dst_cat.src(lower.component(lower.src_functor.src_cat.dst(m)))
            = upper.src_functor.dst_cat.dst(lower.src_functor.mor_map(m))
} by {
    if vertical_composable(upper, lower) {
        vc_lower_component_src_in_upper_cat(upper, lower, lower.src_functor.src_cat.dst(m))
        vc_lower_mor_map_dst(upper, lower, m)
        upper.src_functor.dst_cat.src(lower.component(lower.src_functor.src_cat.dst(m))) = upper.src_functor.dst_cat.dst(lower.src_functor.mor_map(m))
    }
}

/// Composability: upper component at y composable with G_mor(m) in upper's dst_cat.
theorem vc_theta_G_composable[O1, M1, O2, M2](
    upper: NaturalTransformation[O1, M1, O2, M2],
    lower: NaturalTransformation[O1, M1, O2, M2],
    m: M1
) {
    vertical_composable(upper, lower) implies
        upper.src_functor.dst_cat.src(upper.component(lower.src_functor.src_cat.dst(m)))
            = upper.src_functor.dst_cat.dst(upper.src_functor.mor_map(m))
} by {
    if vertical_composable(upper, lower) {
        nat_trans_component_src(upper, lower.src_functor.src_cat.dst(m))
        vc_upper_mor_map_dst(upper, lower, m)
        upper.src_functor.dst_cat.src(upper.component(lower.src_functor.src_cat.dst(m))) = upper.src_functor.dst_cat.dst(upper.src_functor.mor_map(m))
    }
}

/// Composability: G_mor(m) composable with η_x in upper's dst_cat.
theorem vc_G_eta_composable[O1, M1, O2, M2](
    upper: NaturalTransformation[O1, M1, O2, M2],
    lower: NaturalTransformation[O1, M1, O2, M2],
    m: M1
) {
    vertical_composable(upper, lower) implies
        upper.src_functor.dst_cat.src(upper.src_functor.mor_map(m))
            = upper.src_functor.dst_cat.dst(lower.component(lower.src_functor.src_cat.src(m)))
} by {
    if vertical_composable(upper, lower) {
        vc_upper_mor_map_src(upper, lower, m)
        vc_lower_component_dst_in_upper_cat(upper, lower, lower.src_functor.src_cat.src(m))
        upper.src_functor.dst_cat.src(upper.src_functor.mor_map(m)) = upper.src_functor.dst_cat.dst(lower.component(lower.src_functor.src_cat.src(m)))
    }
}

/// Composability: H_mor(m) composable with θ_x in upper's dst_cat.
theorem vc_H_theta_composable[O1, M1, O2, M2](
    upper: NaturalTransformation[O1, M1, O2, M2],
    lower: NaturalTransformation[O1, M1, O2, M2],
    m: M1
) {
    vertical_composable(upper, lower) implies
        upper.src_functor.dst_cat.src(upper.dst_functor.mor_map(m))
            = upper.src_functor.dst_cat.dst(upper.component(lower.src_functor.src_cat.src(m)))
} by {
    if vertical_composable(upper, lower) {
        vc_upper_dst_mor_map_src(upper, lower, m)
        nat_trans_component_dst(upper, lower.src_functor.src_cat.src(m))
        upper.src_functor.dst_cat.src(upper.dst_functor.mor_map(m)) = upper.src_functor.dst_cat.dst(upper.component(lower.src_functor.src_cat.src(m)))
    }
}

/// First associativity step: shifting parens of (θ_y * η_y) * F_mor(m).
theorem vc_assoc_step1[O1, M1, O2, M2](
    upper: NaturalTransformation[O1, M1, O2, M2],
    lower: NaturalTransformation[O1, M1, O2, M2],
    m: M1
) {
    vertical_composable(upper, lower) implies
        upper.src_functor.dst_cat.compose(
            upper.src_functor.dst_cat.compose(upper.component(lower.src_functor.src_cat.dst(m)), lower.component(lower.src_functor.src_cat.dst(m))),
            lower.src_functor.mor_map(m)
        ) = upper.src_functor.dst_cat.compose(
            upper.component(lower.src_functor.src_cat.dst(m)),
            upper.src_functor.dst_cat.compose(lower.component(lower.src_functor.src_cat.dst(m)), lower.src_functor.mor_map(m))
        )
} by {
    if vertical_composable(upper, lower) {
        vertical_compose_components_composable(upper, lower, lower.src_functor.src_cat.dst(m))
        vc_eta_F_composable(upper, lower, m)
        category_compose_assoc(upper.src_functor.dst_cat,
            upper.component(lower.src_functor.src_cat.dst(m)),
            lower.component(lower.src_functor.src_cat.dst(m)),
            lower.src_functor.mor_map(m))
        upper.src_functor.dst_cat.compose(upper.src_functor.dst_cat.compose(upper.component(lower.src_functor.src_cat.dst(m)), lower.component(lower.src_functor.src_cat.dst(m))), lower.src_functor.mor_map(m)) = upper.src_functor.dst_cat.compose(upper.component(lower.src_functor.src_cat.dst(m)), upper.src_functor.dst_cat.compose(lower.component(lower.src_functor.src_cat.dst(m)), lower.src_functor.mor_map(m)))
    }
}

/// Second associativity step (reversed): shifting parens of θ_y * (G_mor(m) * η_x).
theorem vc_assoc_step3[O1, M1, O2, M2](
    upper: NaturalTransformation[O1, M1, O2, M2],
    lower: NaturalTransformation[O1, M1, O2, M2],
    m: M1
) {
    vertical_composable(upper, lower) implies
        upper.src_functor.dst_cat.compose(
            upper.src_functor.dst_cat.compose(upper.component(lower.src_functor.src_cat.dst(m)), upper.src_functor.mor_map(m)),
            lower.component(lower.src_functor.src_cat.src(m))
        ) = upper.src_functor.dst_cat.compose(
            upper.component(lower.src_functor.src_cat.dst(m)),
            upper.src_functor.dst_cat.compose(upper.src_functor.mor_map(m), lower.component(lower.src_functor.src_cat.src(m)))
        )
} by {
    if vertical_composable(upper, lower) {
        vc_theta_G_composable(upper, lower, m)
        vc_G_eta_composable(upper, lower, m)
        category_compose_assoc(upper.src_functor.dst_cat,
            upper.component(lower.src_functor.src_cat.dst(m)),
            upper.src_functor.mor_map(m),
            lower.component(lower.src_functor.src_cat.src(m)))
        upper.src_functor.dst_cat.compose(upper.src_functor.dst_cat.compose(upper.component(lower.src_functor.src_cat.dst(m)), upper.src_functor.mor_map(m)), lower.component(lower.src_functor.src_cat.src(m))) = upper.src_functor.dst_cat.compose(upper.component(lower.src_functor.src_cat.dst(m)), upper.src_functor.dst_cat.compose(upper.src_functor.mor_map(m), lower.component(lower.src_functor.src_cat.src(m))))
    }
}

/// Third associativity step: shifting parens of (H_mor(m) * θ_x) * η_x.
theorem vc_assoc_step5[O1, M1, O2, M2](
    upper: NaturalTransformation[O1, M1, O2, M2],
    lower: NaturalTransformation[O1, M1, O2, M2],
    m: M1
) {
    vertical_composable(upper, lower) implies
        upper.src_functor.dst_cat.compose(
            upper.src_functor.dst_cat.compose(upper.dst_functor.mor_map(m), upper.component(lower.src_functor.src_cat.src(m))),
            lower.component(lower.src_functor.src_cat.src(m))
        ) = upper.src_functor.dst_cat.compose(
            upper.dst_functor.mor_map(m),
            upper.src_functor.dst_cat.compose(upper.component(lower.src_functor.src_cat.src(m)), lower.component(lower.src_functor.src_cat.src(m)))
        )
} by {
    if vertical_composable(upper, lower) {
        vc_H_theta_composable(upper, lower, m)
        vertical_compose_components_composable(upper, lower, lower.src_functor.src_cat.src(m))
        category_compose_assoc(upper.src_functor.dst_cat,
            upper.dst_functor.mor_map(m),
            upper.component(lower.src_functor.src_cat.src(m)),
            lower.component(lower.src_functor.src_cat.src(m)))
        upper.src_functor.dst_cat.compose(upper.src_functor.dst_cat.compose(upper.dst_functor.mor_map(m), upper.component(lower.src_functor.src_cat.src(m))), lower.component(lower.src_functor.src_cat.src(m))) = upper.src_functor.dst_cat.compose(upper.dst_functor.mor_map(m), upper.src_functor.dst_cat.compose(upper.component(lower.src_functor.src_cat.src(m)), lower.component(lower.src_functor.src_cat.src(m))))
    }
}

/// Naturality, block A: LHS chains to a normalized middle form using assoc1 + η nat + assoc3 reversed.
theorem vc_naturality_blockA[O1, M1, O2, M2](
    upper: NaturalTransformation[O1, M1, O2, M2],
    lower: NaturalTransformation[O1, M1, O2, M2],
    m: M1
) {
    vertical_composable(upper, lower) implies
        upper.src_functor.dst_cat.compose(upper.src_functor.dst_cat.compose(upper.component(lower.src_functor.src_cat.dst(m)), lower.component(lower.src_functor.src_cat.dst(m))), lower.src_functor.mor_map(m))
            = upper.src_functor.dst_cat.compose(upper.src_functor.dst_cat.compose(upper.component(lower.src_functor.src_cat.dst(m)), upper.src_functor.mor_map(m)), lower.component(lower.src_functor.src_cat.src(m)))
} by {
    if vertical_composable(upper, lower) {
        vc_assoc_step1(upper, lower, m)
        vc_eta_naturality(upper, lower, m)
        vc_assoc_step3(upper, lower, m)
        upper.src_functor.dst_cat.compose(upper.src_functor.dst_cat.compose(upper.component(lower.src_functor.src_cat.dst(m)), lower.component(lower.src_functor.src_cat.dst(m))), lower.src_functor.mor_map(m)) = upper.src_functor.dst_cat.compose(upper.src_functor.dst_cat.compose(upper.component(lower.src_functor.src_cat.dst(m)), upper.src_functor.mor_map(m)), lower.component(lower.src_functor.src_cat.src(m)))
    }
}

/// Naturality, block B: middle form chains to RHS form using θ nat + assoc5.
theorem vc_naturality_blockB[O1, M1, O2, M2](
    upper: NaturalTransformation[O1, M1, O2, M2],
    lower: NaturalTransformation[O1, M1, O2, M2],
    m: M1
) {
    vertical_composable(upper, lower) implies
        upper.src_functor.dst_cat.compose(upper.src_functor.dst_cat.compose(upper.component(lower.src_functor.src_cat.dst(m)), upper.src_functor.mor_map(m)), lower.component(lower.src_functor.src_cat.src(m)))
            = upper.src_functor.dst_cat.compose(upper.dst_functor.mor_map(m), upper.src_functor.dst_cat.compose(upper.component(lower.src_functor.src_cat.src(m)), lower.component(lower.src_functor.src_cat.src(m))))
} by {
    if vertical_composable(upper, lower) {
        vc_theta_naturality(upper, lower, m)
        vc_assoc_step5(upper, lower, m)
        upper.src_functor.dst_cat.compose(upper.src_functor.dst_cat.compose(upper.component(lower.src_functor.src_cat.dst(m)), upper.src_functor.mor_map(m)), lower.component(lower.src_functor.src_cat.src(m))) = upper.src_functor.dst_cat.compose(upper.dst_functor.mor_map(m), upper.src_functor.dst_cat.compose(upper.component(lower.src_functor.src_cat.src(m)), lower.component(lower.src_functor.src_cat.src(m))))
    }
}

/// Combine LHS unfold with blockA chain.
theorem vc_naturality_lhs_to_blockA_end[O1, M1, O2, M2](
    upper: NaturalTransformation[O1, M1, O2, M2],
    lower: NaturalTransformation[O1, M1, O2, M2],
    m: M1
) {
    vertical_composable(upper, lower) implies
        upper.src_functor.dst_cat.compose(vertical_compose_component(upper, lower, lower.src_functor.src_cat.dst(m)), lower.src_functor.mor_map(m))
            = upper.src_functor.dst_cat.compose(upper.src_functor.dst_cat.compose(upper.component(lower.src_functor.src_cat.dst(m)), upper.src_functor.mor_map(m)), lower.component(lower.src_functor.src_cat.src(m)))
} by {
    if vertical_composable(upper, lower) {
        upper.src_functor.dst_cat.compose(vertical_compose_component(upper, lower, lower.src_functor.src_cat.dst(m)), lower.src_functor.mor_map(m)) = upper.src_functor.dst_cat.compose(upper.src_functor.dst_cat.compose(upper.component(lower.src_functor.src_cat.dst(m)), lower.component(lower.src_functor.src_cat.dst(m))), lower.src_functor.mor_map(m))
        vc_naturality_blockA(upper, lower, m)
    }
}

/// Combine blockB end with RHS fold.
theorem vc_naturality_blockB_to_rhs[O1, M1, O2, M2](
    upper: NaturalTransformation[O1, M1, O2, M2],
    lower: NaturalTransformation[O1, M1, O2, M2],
    m: M1
) {
    vertical_composable(upper, lower) implies
        upper.src_functor.dst_cat.compose(upper.src_functor.dst_cat.compose(upper.component(lower.src_functor.src_cat.dst(m)), upper.src_functor.mor_map(m)), lower.component(lower.src_functor.src_cat.src(m)))
            = upper.src_functor.dst_cat.compose(upper.dst_functor.mor_map(m), vertical_compose_component(upper, lower, lower.src_functor.src_cat.src(m)))
} by {
    if vertical_composable(upper, lower) {
        vc_naturality_blockB(upper, lower, m)
        upper.src_functor.dst_cat.compose(upper.src_functor.dst_cat.compose(upper.component(lower.src_functor.src_cat.dst(m)), upper.src_functor.mor_map(m)), lower.component(lower.src_functor.src_cat.src(m))) = upper.src_functor.dst_cat.compose(upper.dst_functor.mor_map(m), vertical_compose_component(upper, lower, lower.src_functor.src_cat.src(m)))
    }
}

/// Vertical composition naturality in upper's target category, before bridging to lower's frame.
theorem vc_naturality_upper_cat[O1, M1, O2, M2](
    upper: NaturalTransformation[O1, M1, O2, M2],
    lower: NaturalTransformation[O1, M1, O2, M2],
    m: M1
) {
    vertical_composable(upper, lower) implies
        upper.src_functor.dst_cat.compose(vertical_compose_component(upper, lower, lower.src_functor.src_cat.dst(m)), lower.src_functor.mor_map(m))
            = upper.src_functor.dst_cat.compose(upper.dst_functor.mor_map(m), vertical_compose_component(upper, lower, lower.src_functor.src_cat.src(m)))
} by {
    if vertical_composable(upper, lower) {
        vc_naturality_lhs_to_blockA_end(upper, lower, m)
        vc_naturality_blockB_to_rhs(upper, lower, m)
        upper.src_functor.dst_cat.compose(vertical_compose_component(upper, lower, lower.src_functor.src_cat.dst(m)), lower.src_functor.mor_map(m)) = upper.src_functor.dst_cat.compose(upper.dst_functor.mor_map(m), vertical_compose_component(upper, lower, lower.src_functor.src_cat.src(m)))
    }
}

/// The vertical compose component map satisfies the naturality axiom.
theorem vc_naturality_axiom[O1, M1, O2, M2](
    upper: NaturalTransformation[O1, M1, O2, M2],
    lower: NaturalTransformation[O1, M1, O2, M2]
) {
    vertical_composable(upper, lower) implies
        nat_trans_naturality_axiom(lower.src_functor, upper.dst_functor, vertical_compose_component(upper, lower))
} by {
    if vertical_composable(upper, lower) {
        vc_dst_cat_eq(upper, lower)
        forall(m: M1) {
            vc_naturality_upper_cat(upper, lower, m)
            vertical_compose_component(upper, lower)(lower.src_functor.src_cat.dst(m)) = vertical_compose_component(upper, lower, lower.src_functor.src_cat.dst(m))
            vertical_compose_component(upper, lower)(lower.src_functor.src_cat.src(m)) = vertical_compose_component(upper, lower, lower.src_functor.src_cat.src(m))
            lower.src_functor.dst_cat.compose(vertical_compose_component(upper, lower)(lower.src_functor.src_cat.dst(m)), lower.src_functor.mor_map(m)) = lower.src_functor.dst_cat.compose(upper.dst_functor.mor_map(m), vertical_compose_component(upper, lower)(lower.src_functor.src_cat.src(m)))
        }
    }
}

/// The vertical compose component map is a natural transformation from lower's source functor to upper's target functor.
theorem vc_is_natural_transformation[O1, M1, O2, M2](
    upper: NaturalTransformation[O1, M1, O2, M2],
    lower: NaturalTransformation[O1, M1, O2, M2]
) {
    vertical_composable(upper, lower) implies
        is_natural_transformation(lower.src_functor, upper.dst_functor, vertical_compose_component(upper, lower))
} by {
    if vertical_composable(upper, lower) {
        vertical_composable_src_cat(upper, lower)
        vertical_composable_dst_cat(upper, lower)
        functors_parallel(lower.src_functor, upper.dst_functor)
        vc_endpoints_axiom(upper, lower)
        vc_naturality_axiom(upper, lower)
        is_natural_transformation(lower.src_functor, upper.dst_functor, vertical_compose_component(upper, lower))
    }
}

/// The vertical composition of two natural transformations, packaged as an optional natural transformation.
define vertical_compose_nat_trans[O1, M1, O2, M2](
    upper: NaturalTransformation[O1, M1, O2, M2],
    lower: NaturalTransformation[O1, M1, O2, M2]
) -> Option[NaturalTransformation[O1, M1, O2, M2]] {
    NaturalTransformation[O1, M1, O2, M2].new(
        lower.src_functor, upper.dst_functor, vertical_compose_component(upper, lower)
    )
}

/// The vertical composition of two vertically composable natural transformations is some bundled transformation.
theorem vertical_compose_nat_trans_some[O1, M1, O2, M2](
    upper: NaturalTransformation[O1, M1, O2, M2],
    lower: NaturalTransformation[O1, M1, O2, M2]
) {
    vertical_composable(upper, lower) implies exists(n: NaturalTransformation[O1, M1, O2, M2]) {
        vertical_compose_nat_trans(upper, lower) = Option.some(n)
    }
} by {
    if vertical_composable(upper, lower) {
        vc_is_natural_transformation(upper, lower)
        let n: NaturalTransformation[O1, M1, O2, M2] satisfy {
            NaturalTransformation[O1, M1, O2, M2].new(lower.src_functor, upper.dst_functor, vertical_compose_component(upper, lower)) = Option.some(n)
        }
        vertical_compose_nat_trans(upper, lower) = Option.some(n)
    }
}

/// The bundled vertical composition has lower's source functor as its source.
theorem vertical_compose_nat_trans_src_functor[O1, M1, O2, M2](
    upper: NaturalTransformation[O1, M1, O2, M2],
    lower: NaturalTransformation[O1, M1, O2, M2],
    n: NaturalTransformation[O1, M1, O2, M2]
) {
    vertical_compose_nat_trans(upper, lower) = Option.some(n) implies n.src_functor = lower.src_functor
} by {
    if vertical_compose_nat_trans(upper, lower) = Option.some(n) {
        NaturalTransformation[O1, M1, O2, M2].new(lower.src_functor, upper.dst_functor, vertical_compose_component(upper, lower)) = Option.some(n)
        nat_trans_new_src_functor(lower.src_functor, upper.dst_functor, vertical_compose_component(upper, lower), n)
    }
}

/// The bundled vertical composition has upper's target functor as its target.
theorem vertical_compose_nat_trans_dst_functor[O1, M1, O2, M2](
    upper: NaturalTransformation[O1, M1, O2, M2],
    lower: NaturalTransformation[O1, M1, O2, M2],
    n: NaturalTransformation[O1, M1, O2, M2]
) {
    vertical_compose_nat_trans(upper, lower) = Option.some(n) implies n.dst_functor = upper.dst_functor
} by {
    if vertical_compose_nat_trans(upper, lower) = Option.some(n) {
        NaturalTransformation[O1, M1, O2, M2].new(lower.src_functor, upper.dst_functor, vertical_compose_component(upper, lower)) = Option.some(n)
        nat_trans_new_dst_functor(lower.src_functor, upper.dst_functor, vertical_compose_component(upper, lower), n)
    }
}

/// The bundled vertical composition has the pointwise composed component map.
theorem vertical_compose_nat_trans_component[O1, M1, O2, M2](
    upper: NaturalTransformation[O1, M1, O2, M2],
    lower: NaturalTransformation[O1, M1, O2, M2],
    n: NaturalTransformation[O1, M1, O2, M2]
) {
    vertical_compose_nat_trans(upper, lower) = Option.some(n) implies n.component = vertical_compose_component(upper, lower)
} by {
    if vertical_compose_nat_trans(upper, lower) = Option.some(n) {
        NaturalTransformation[O1, M1, O2, M2].new(lower.src_functor, upper.dst_functor, vertical_compose_component(upper, lower)) = Option.some(n)
        nat_trans_new_component(lower.src_functor, upper.dst_functor, vertical_compose_component(upper, lower), n)
    }
}

/// At each object, vertical composition of three composable natural transformations is associative.
theorem vertical_compose_component_assoc_at[O1, M1, O2, M2](
    top: NaturalTransformation[O1, M1, O2, M2],
    middle: NaturalTransformation[O1, M1, O2, M2],
    bottom: NaturalTransformation[O1, M1, O2, M2],
    x: O1
) {
    vertical_composable(top, middle) and vertical_composable(middle, bottom) implies
        top.src_functor.dst_cat.compose(vertical_compose_component(top, middle, x), bottom.component(x)) =
        top.src_functor.dst_cat.compose(top.component(x), vertical_compose_component(middle, bottom, x))
} by {
    if vertical_composable(top, middle) and vertical_composable(middle, bottom) {
        vertical_compose_components_composable(top, middle, x)
        vertical_compose_components_composable(middle, bottom, x)
        vc_dst_cat_eq(top, middle)
        category_compose_assoc(top.src_functor.dst_cat, top.component(x), middle.component(x), bottom.component(x))
        top.src_functor.dst_cat.compose(
            top.src_functor.dst_cat.compose(top.component(x), middle.component(x)),
            bottom.component(x)
        ) = top.src_functor.dst_cat.compose(
            top.component(x),
            top.src_functor.dst_cat.compose(middle.component(x), bottom.component(x))
        )
        top.src_functor.dst_cat.compose(vertical_compose_component(top, middle, x), bottom.component(x)) =
            top.src_functor.dst_cat.compose(top.component(x), vertical_compose_component(middle, bottom, x))
    }
}

/// Bundled vertical composition is associative for explicitly supplied composite witnesses.
theorem vertical_compose_nat_trans_assoc[O1, M1, O2, M2](
    top: NaturalTransformation[O1, M1, O2, M2],
    middle: NaturalTransformation[O1, M1, O2, M2],
    bottom: NaturalTransformation[O1, M1, O2, M2],
    top_middle: NaturalTransformation[O1, M1, O2, M2],
    middle_bottom: NaturalTransformation[O1, M1, O2, M2],
    left: NaturalTransformation[O1, M1, O2, M2],
    right: NaturalTransformation[O1, M1, O2, M2]
) {
    vertical_composable(top, middle) and vertical_composable(middle, bottom) and
    vertical_compose_nat_trans(top, middle) = Option.some(top_middle) and
    vertical_compose_nat_trans(middle, bottom) = Option.some(middle_bottom) and
    vertical_compose_nat_trans(top_middle, bottom) = Option.some(left) and
    vertical_compose_nat_trans(top, middle_bottom) = Option.some(right)
    implies left = right
} by {
    if vertical_composable(top, middle) and vertical_composable(middle, bottom) and
        vertical_compose_nat_trans(top, middle) = Option.some(top_middle) and
        vertical_compose_nat_trans(middle, bottom) = Option.some(middle_bottom) and
        vertical_compose_nat_trans(top_middle, bottom) = Option.some(left) and
        vertical_compose_nat_trans(top, middle_bottom) = Option.some(right) {
        vertical_compose_nat_trans_src_functor(top, middle, top_middle)
        vertical_compose_nat_trans_dst_functor(top, middle, top_middle)
        vertical_compose_nat_trans_component(top, middle, top_middle)
        top_middle.src_functor = middle.src_functor
        top_middle.dst_functor = top.dst_functor
        top_middle.component = vertical_compose_component(top, middle)

        vertical_compose_nat_trans_src_functor(middle, bottom, middle_bottom)
        vertical_compose_nat_trans_dst_functor(middle, bottom, middle_bottom)
        vertical_compose_nat_trans_component(middle, bottom, middle_bottom)
        middle_bottom.src_functor = bottom.src_functor
        middle_bottom.dst_functor = middle.dst_functor
        middle_bottom.component = vertical_compose_component(middle, bottom)

        vertical_compose_nat_trans_src_functor(top_middle, bottom, left)
        vertical_compose_nat_trans_dst_functor(top_middle, bottom, left)
        vertical_compose_nat_trans_component(top_middle, bottom, left)
        left.src_functor = bottom.src_functor
        left.dst_functor = top_middle.dst_functor
        left.dst_functor = top.dst_functor
        left.component = vertical_compose_component(top_middle, bottom)

        vertical_compose_nat_trans_src_functor(top, middle_bottom, right)
        vertical_compose_nat_trans_dst_functor(top, middle_bottom, right)
        vertical_compose_nat_trans_component(top, middle_bottom, right)
        right.src_functor = middle_bottom.src_functor
        right.src_functor = bottom.src_functor
        right.dst_functor = top.dst_functor
        right.component = vertical_compose_component(top, middle_bottom)

        forall(x: O1) {
            vertical_compose_component(top_middle, bottom)(x) = vertical_compose_component(top_middle, bottom, x)
            vertical_compose_component(top_middle, bottom, x) =
                top_middle.src_functor.dst_cat.compose(top_middle.component(x), bottom.component(x))
            vc_dst_cat_eq(top, middle)
            vertical_compose_component(top, middle)(x) = vertical_compose_component(top, middle, x)

            vertical_compose_component(top, middle_bottom)(x) = vertical_compose_component(top, middle_bottom, x)
            vertical_compose_component(top, middle_bottom, x) =
                top.src_functor.dst_cat.compose(top.component(x), middle_bottom.component(x))
            vertical_compose_component(middle, bottom)(x) = vertical_compose_component(middle, bottom, x)

            vertical_compose_component_assoc_at(top, middle, bottom, x)
            top.src_functor.dst_cat.compose(vertical_compose_component(top, middle, x), bottom.component(x)) =
                top.src_functor.dst_cat.compose(top.component(x), vertical_compose_component(middle, bottom, x))
            left.component(x) = right.component(x)
        }
        left.component = right.component
        left = right
    }
}

/// True if the outer natural transformation can be horizontally composed with the inner natural transformation.
define horizontal_composable[O1, M1, O2, M2, O3, M3](
    outer: NaturalTransformation[O2, M2, O3, M3],
    inner: NaturalTransformation[O1, M1, O2, M2]
) -> Bool {
    outer.src_functor.src_cat = inner.src_functor.dst_cat
}

/// The component morphism of the horizontal composition of two natural transformations at an object.
define horizontal_compose_component[O1, M1, O2, M2, O3, M3](
    outer: NaturalTransformation[O2, M2, O3, M3],
    inner: NaturalTransformation[O1, M1, O2, M2],
    x: O1
) -> M3 {
    outer.src_functor.dst_cat.compose(
        outer.component(inner.dst_functor.obj_map(x)),
        outer.src_functor.mor_map(inner.component(x))
    )
}

/// In a horizontal composition, the outer source functor can act on the inner component at an object.
theorem horizontal_compose_components_composable[O1, M1, O2, M2, O3, M3](
    outer: NaturalTransformation[O2, M2, O3, M3],
    inner: NaturalTransformation[O1, M1, O2, M2],
    x: O1
) {
    horizontal_composable(outer, inner) implies
        outer.src_functor.dst_cat.src(outer.component(inner.dst_functor.obj_map(x)))
            = outer.src_functor.dst_cat.dst(outer.src_functor.mor_map(inner.component(x)))
} by {
    if horizontal_composable(outer, inner) {
        nat_trans_component_src(outer, inner.dst_functor.obj_map(x))
        functor_dst(outer.src_functor, inner.component(x))
        outer.src_functor.src_cat.dst(inner.component(x)) = inner.src_functor.dst_cat.dst(inner.component(x))
        nat_trans_component_dst(inner, x)
        outer.src_functor.dst_cat.src(outer.component(inner.dst_functor.obj_map(x))) = outer.src_functor.dst_cat.dst(outer.src_functor.mor_map(inner.component(x)))
    }
}

/// The source endpoint of a horizontal composition component in the outer target category.
theorem horizontal_compose_component_src[O1, M1, O2, M2, O3, M3](
    outer: NaturalTransformation[O2, M2, O3, M3],
    inner: NaturalTransformation[O1, M1, O2, M2],
    src_comp: Functor[O1, M1, O3, M3],
    x: O1
) {
    horizontal_composable(outer, inner)
    and compose_functor(outer.src_functor, inner.src_functor) = Option.some(src_comp)
    implies outer.src_functor.dst_cat.src(horizontal_compose_component(outer, inner, x)) = src_comp.obj_map(x)
} by {
    if horizontal_composable(outer, inner)
    and compose_functor(outer.src_functor, inner.src_functor) = Option.some(src_comp) {
        horizontal_compose_components_composable(outer, inner, x)
        category_compose_src(outer.src_functor.dst_cat,
            outer.component(inner.dst_functor.obj_map(x)),
            outer.src_functor.mor_map(inner.component(x)))
        functor_src(outer.src_functor, inner.component(x))
        outer.src_functor.src_cat.src(inner.component(x)) = inner.src_functor.dst_cat.src(inner.component(x))
        nat_trans_component_src(inner, x)
        compose_functor_obj_map(outer.src_functor, inner.src_functor, src_comp)
        compose(outer.src_functor.obj_map, inner.src_functor.obj_map, x) = outer.src_functor.obj_map(inner.src_functor.obj_map(x))
        outer.src_functor.dst_cat.src(horizontal_compose_component(outer, inner, x)) = src_comp.obj_map(x)
    }
}

/// The target endpoint of a horizontal composition component in the outer target category.
theorem horizontal_compose_component_dst[O1, M1, O2, M2, O3, M3](
    outer: NaturalTransformation[O2, M2, O3, M3],
    inner: NaturalTransformation[O1, M1, O2, M2],
    dst_comp: Functor[O1, M1, O3, M3],
    x: O1
) {
    horizontal_composable(outer, inner)
    and compose_functor(outer.dst_functor, inner.dst_functor) = Option.some(dst_comp)
    implies outer.src_functor.dst_cat.dst(horizontal_compose_component(outer, inner, x)) = dst_comp.obj_map(x)
} by {
    if horizontal_composable(outer, inner)
    and compose_functor(outer.dst_functor, inner.dst_functor) = Option.some(dst_comp) {
        horizontal_compose_components_composable(outer, inner, x)
        category_compose_dst(outer.src_functor.dst_cat,
            outer.component(inner.dst_functor.obj_map(x)),
            outer.src_functor.mor_map(inner.component(x)))
        nat_trans_component_dst(outer, inner.dst_functor.obj_map(x))
        compose_functor_obj_map(outer.dst_functor, inner.dst_functor, dst_comp)
        compose(outer.dst_functor.obj_map, inner.dst_functor.obj_map, x) = outer.dst_functor.obj_map(inner.dst_functor.obj_map(x))
        outer.src_functor.dst_cat.dst(horizontal_compose_component(outer, inner, x)) = dst_comp.obj_map(x)
    }
}

/// The horizontal composition of two natural transformations, packaged relative to chosen composite functors.
define horizontal_compose_nat_trans[O1, M1, O2, M2, O3, M3](
    outer: NaturalTransformation[O2, M2, O3, M3],
    inner: NaturalTransformation[O1, M1, O2, M2],
    src_comp: Functor[O1, M1, O3, M3],
    dst_comp: Functor[O1, M1, O3, M3]
) -> Option[NaturalTransformation[O1, M1, O3, M3]] {
    NaturalTransformation[O1, M1, O3, M3].new(
        src_comp, dst_comp, horizontal_compose_component(outer, inner)
    )
}

/// A bundled horizontal composition has the chosen source composite as its source.
theorem horizontal_compose_nat_trans_src_functor[O1, M1, O2, M2, O3, M3](
    outer: NaturalTransformation[O2, M2, O3, M3],
    inner: NaturalTransformation[O1, M1, O2, M2],
    src_comp: Functor[O1, M1, O3, M3],
    dst_comp: Functor[O1, M1, O3, M3],
    n: NaturalTransformation[O1, M1, O3, M3]
) {
    horizontal_compose_nat_trans(outer, inner, src_comp, dst_comp) = Option.some(n)
    implies n.src_functor = src_comp
} by {
    if horizontal_compose_nat_trans(outer, inner, src_comp, dst_comp) = Option.some(n) {
        NaturalTransformation[O1, M1, O3, M3].new(
            src_comp, dst_comp, horizontal_compose_component(outer, inner)
        ) = Option.some(n)
        nat_trans_new_src_functor(src_comp, dst_comp, horizontal_compose_component(outer, inner), n)
    }
}

/// A bundled horizontal composition has the chosen target composite as its target.
theorem horizontal_compose_nat_trans_dst_functor[O1, M1, O2, M2, O3, M3](
    outer: NaturalTransformation[O2, M2, O3, M3],
    inner: NaturalTransformation[O1, M1, O2, M2],
    src_comp: Functor[O1, M1, O3, M3],
    dst_comp: Functor[O1, M1, O3, M3],
    n: NaturalTransformation[O1, M1, O3, M3]
) {
    horizontal_compose_nat_trans(outer, inner, src_comp, dst_comp) = Option.some(n)
    implies n.dst_functor = dst_comp
} by {
    if horizontal_compose_nat_trans(outer, inner, src_comp, dst_comp) = Option.some(n) {
        NaturalTransformation[O1, M1, O3, M3].new(
            src_comp, dst_comp, horizontal_compose_component(outer, inner)
        ) = Option.some(n)
        nat_trans_new_dst_functor(src_comp, dst_comp, horizontal_compose_component(outer, inner), n)
    }
}

/// A bundled horizontal composition has the standard component map.
theorem horizontal_compose_nat_trans_component[O1, M1, O2, M2, O3, M3](
    outer: NaturalTransformation[O2, M2, O3, M3],
    inner: NaturalTransformation[O1, M1, O2, M2],
    src_comp: Functor[O1, M1, O3, M3],
    dst_comp: Functor[O1, M1, O3, M3],
    n: NaturalTransformation[O1, M1, O3, M3]
) {
    horizontal_compose_nat_trans(outer, inner, src_comp, dst_comp) = Option.some(n)
    implies n.component = horizontal_compose_component(outer, inner)
} by {
    if horizontal_compose_nat_trans(outer, inner, src_comp, dst_comp) = Option.some(n) {
        NaturalTransformation[O1, M1, O3, M3].new(
            src_comp, dst_comp, horizontal_compose_component(outer, inner)
        ) = Option.some(n)
        nat_trans_new_component(src_comp, dst_comp, horizontal_compose_component(outer, inner), n)
    }
}

/// Horizontally composing an identity natural transformation on the left gives left whiskering on components.
theorem horizontal_compose_identity_outer_component_at[O1, M1, O2, M2, O3, M3](
    h: Functor[O2, M2, O3, M3],
    a: NaturalTransformation[O1, M1, O2, M2],
    x: O1
) {
    horizontal_composable(identity_nat_trans(h), a) implies
        horizontal_compose_component(identity_nat_trans(h), a, x) = h.mor_map(a.component(x))
} by {
    let id_h = identity_nat_trans(h)
    if horizontal_composable(identity_nat_trans(h), a) {
        horizontal_composable(id_h, a)
        identity_nat_trans_src_functor(h)
        id_h.src_functor = h
        identity_nat_trans_component_eq(h)
        id_h.component = identity_nat_trans_component(h)
        identity_nat_trans_component(h)(a.dst_functor.obj_map(x)) =
            identity_nat_trans_component(h, a.dst_functor.obj_map(x))
        horizontal_compose_component(id_h, a, x) = h.dst_cat.compose(
            h.dst_cat.identity(h.obj_map(a.dst_functor.obj_map(x))),
            h.mor_map(a.component(x))
        )

        functor_dst(h, a.component(x))
        h.src_cat.dst = a.src_functor.dst_cat.dst
        nat_trans_component_dst(a, x)
        category_id_left(h.dst_cat, h.mor_map(a.component(x)))
        horizontal_compose_component(id_h, a, x) = h.mor_map(a.component(x))
    }
}

/// The bundled horizontal composition with an identity natural transformation on the left has left-whiskered components.
theorem horizontal_compose_identity_outer_nat_trans_component[O1, M1, O2, M2, O3, M3](
    h: Functor[O2, M2, O3, M3],
    a: NaturalTransformation[O1, M1, O2, M2],
    src_comp: Functor[O1, M1, O3, M3],
    dst_comp: Functor[O1, M1, O3, M3],
    n: NaturalTransformation[O1, M1, O3, M3]
) {
    horizontal_composable(identity_nat_trans(h), a) and
    compose_functor(h, a.src_functor) = Option.some(src_comp) and
    compose_functor(h, a.dst_functor) = Option.some(dst_comp) and
    horizontal_compose_nat_trans(identity_nat_trans(h), a, src_comp, dst_comp) = Option.some(n)
    implies n.component = function(x: O1) { h.mor_map(a.component(x)) }
} by {
    let id_h = identity_nat_trans(h)
    if horizontal_composable(identity_nat_trans(h), a) and
        compose_functor(h, a.src_functor) = Option.some(src_comp) and
        compose_functor(h, a.dst_functor) = Option.some(dst_comp) and
        horizontal_compose_nat_trans(identity_nat_trans(h), a, src_comp, dst_comp) = Option.some(n) {
        horizontal_compose_nat_trans_component(id_h, a, src_comp, dst_comp, n)
        n.component = horizontal_compose_component(id_h, a)
        forall(x: O1) {
            horizontal_compose_identity_outer_component_at(h, a, x)
            horizontal_compose_component(id_h, a)(x) = horizontal_compose_component(id_h, a, x)
            (function(y: O1) { h.mor_map(a.component(y)) })(x) = h.mor_map(a.component(x))
            n.component(x) = (function(y: O1) { h.mor_map(a.component(y)) })(x)
        }
        n.component = function(y: O1) { h.mor_map(a.component(y)) }
    }
}

/// Horizontally composing with an identity natural transformation on the right gives right whiskering on components.
theorem horizontal_compose_identity_inner_component_at[O1, M1, O2, M2, O3, M3](
    a: NaturalTransformation[O2, M2, O3, M3],
    h: Functor[O1, M1, O2, M2],
    x: O1
) {
    horizontal_composable(a, identity_nat_trans(h)) implies
        horizontal_compose_component(a, identity_nat_trans(h), x) = a.component(h.obj_map(x))
} by {
    let id_h = identity_nat_trans(h)
    if horizontal_composable(a, identity_nat_trans(h)) {
        horizontal_composable(a, id_h)
        identity_nat_trans_src_functor(h)
        id_h.src_functor = h
        identity_nat_trans_dst_functor(h)
        id_h.dst_functor = h
        identity_nat_trans_component_eq(h)
        id_h.component = identity_nat_trans_component(h)
        identity_nat_trans_component(h)(x) = identity_nat_trans_component(h, x)

        a.src_functor.src_cat.identity(h.obj_map(x)) = h.dst_cat.identity(h.obj_map(x))
        functor_identity(a.src_functor, h.obj_map(x))

        nat_trans_component_src(a, h.obj_map(x))
        category_id_right(a.src_functor.dst_cat, a.component(h.obj_map(x)))
        horizontal_compose_component(a, id_h, x) = a.component(h.obj_map(x))
    }
}

/// The bundled horizontal composition with an identity natural transformation on the right has right-whiskered components.
theorem horizontal_compose_identity_inner_nat_trans_component[O1, M1, O2, M2, O3, M3](
    a: NaturalTransformation[O2, M2, O3, M3],
    h: Functor[O1, M1, O2, M2],
    src_comp: Functor[O1, M1, O3, M3],
    dst_comp: Functor[O1, M1, O3, M3],
    n: NaturalTransformation[O1, M1, O3, M3]
) {
    horizontal_composable(a, identity_nat_trans(h)) and
    compose_functor(a.src_functor, h) = Option.some(src_comp) and
    compose_functor(a.dst_functor, h) = Option.some(dst_comp) and
    horizontal_compose_nat_trans(a, identity_nat_trans(h), src_comp, dst_comp) = Option.some(n)
    implies n.component = function(x: O1) { a.component(h.obj_map(x)) }
} by {
    let id_h = identity_nat_trans(h)
    if horizontal_composable(a, identity_nat_trans(h)) and
        compose_functor(a.src_functor, h) = Option.some(src_comp) and
        compose_functor(a.dst_functor, h) = Option.some(dst_comp) and
        horizontal_compose_nat_trans(a, identity_nat_trans(h), src_comp, dst_comp) = Option.some(n) {
        horizontal_compose_nat_trans_component(a, id_h, src_comp, dst_comp, n)
        n.component = horizontal_compose_component(a, id_h)
        forall(x: O1) {
            horizontal_compose_identity_inner_component_at(a, h, x)
            horizontal_compose_component(a, id_h)(x) = horizontal_compose_component(a, id_h, x)
            (function(y: O1) { a.component(h.obj_map(y)) })(x) = a.component(h.obj_map(x))
            n.component(x) = (function(y: O1) { a.component(h.obj_map(y)) })(x)
        }
        n.component = function(y: O1) { a.component(h.obj_map(y)) }
    }
}

/// In a horizontal composition, the outer source functor applied to the inner component at the target of `m`
/// is composable with `outer.src_functor.mor_map(inner.src_functor.mor_map(m))`.
theorem hc_F_beta_F_H_composable[O1, M1, O2, M2, O3, M3](
    outer: NaturalTransformation[O2, M2, O3, M3],
    inner: NaturalTransformation[O1, M1, O2, M2],
    m: M1
) {
    horizontal_composable(outer, inner) implies
        outer.src_functor.dst_cat.src(outer.src_functor.mor_map(inner.component(inner.src_functor.src_cat.dst(m))))
            = outer.src_functor.dst_cat.dst(outer.src_functor.mor_map(inner.src_functor.mor_map(m)))
} by {
    if horizontal_composable(outer, inner) {
        functor_src(outer.src_functor, inner.component(inner.src_functor.src_cat.dst(m)))
        outer.src_functor.src_cat.src = inner.src_functor.dst_cat.src
        nat_trans_component_src(inner, inner.src_functor.src_cat.dst(m))
        functor_dst(outer.src_functor, inner.src_functor.mor_map(m))
        outer.src_functor.src_cat.dst = inner.src_functor.dst_cat.dst
        functor_dst(inner.src_functor, m)
        outer.src_functor.dst_cat.src(outer.src_functor.mor_map(inner.component(inner.src_functor.src_cat.dst(m)))) = outer.src_functor.dst_cat.dst(outer.src_functor.mor_map(inner.src_functor.mor_map(m)))
    }
}

/// In a horizontal composition, the outer source functor applied to `inner.dst_functor.mor_map(m)`
/// is composable with the outer source functor applied to the inner component at the source of `m`.
theorem hc_F_K_F_beta_composable[O1, M1, O2, M2, O3, M3](
    outer: NaturalTransformation[O2, M2, O3, M3],
    inner: NaturalTransformation[O1, M1, O2, M2],
    m: M1
) {
    horizontal_composable(outer, inner) implies
        outer.src_functor.dst_cat.src(outer.src_functor.mor_map(inner.dst_functor.mor_map(m)))
            = outer.src_functor.dst_cat.dst(outer.src_functor.mor_map(inner.component(inner.src_functor.src_cat.src(m))))
} by {
    if horizontal_composable(outer, inner) {
        functor_src(outer.src_functor, inner.dst_functor.mor_map(m))
        outer.src_functor.src_cat.src = inner.src_functor.dst_cat.src
        nat_trans_shared_src_cat(inner)
        nat_trans_shared_dst_cat(inner)
        functor_src(inner.dst_functor, m)
        functor_dst(outer.src_functor, inner.component(inner.src_functor.src_cat.src(m)))
        outer.src_functor.src_cat.dst = inner.src_functor.dst_cat.dst
        nat_trans_component_dst(inner, inner.src_functor.src_cat.src(m))
        outer.src_functor.dst_cat.src(outer.src_functor.mor_map(inner.dst_functor.mor_map(m))) = outer.src_functor.dst_cat.dst(outer.src_functor.mor_map(inner.component(inner.src_functor.src_cat.src(m))))
    }
}

/// Outer source functor preserves the inner naturality square.
theorem hc_inner_nat_lifted[O1, M1, O2, M2, O3, M3](
    outer: NaturalTransformation[O2, M2, O3, M3],
    inner: NaturalTransformation[O1, M1, O2, M2],
    m: M1
) {
    horizontal_composable(outer, inner) implies
        outer.src_functor.dst_cat.compose(
            outer.src_functor.mor_map(inner.component(inner.src_functor.src_cat.dst(m))),
            outer.src_functor.mor_map(inner.src_functor.mor_map(m))
        ) = outer.src_functor.dst_cat.compose(
            outer.src_functor.mor_map(inner.dst_functor.mor_map(m)),
            outer.src_functor.mor_map(inner.component(inner.src_functor.src_cat.src(m)))
        )
} by {
    if horizontal_composable(outer, inner) {
        // F functoriality on the LHS pair
        nat_trans_component_src(inner, inner.src_functor.src_cat.dst(m))
        functor_dst(inner.src_functor, m)
        outer.src_functor.src_cat.src = inner.src_functor.dst_cat.src
        outer.src_functor.src_cat.dst = inner.src_functor.dst_cat.dst
        outer.src_functor.src_cat.compose = inner.src_functor.dst_cat.compose
        functor_compose(outer.src_functor,
            inner.component(inner.src_functor.src_cat.dst(m)),
            inner.src_functor.mor_map(m))
        outer.src_functor.mor_map(outer.src_functor.src_cat.compose(
            inner.component(inner.src_functor.src_cat.dst(m)),
            inner.src_functor.mor_map(m))) = outer.src_functor.dst_cat.compose(
                outer.src_functor.mor_map(inner.component(inner.src_functor.src_cat.dst(m))),
                outer.src_functor.mor_map(inner.src_functor.mor_map(m)))
        // inner naturality lifted into B (via cat equality)
        nat_trans_naturality(inner, m)
        // F functoriality on the RHS pair
        functor_dst(inner.dst_functor, m)
        nat_trans_shared_src_cat(inner)
        nat_trans_shared_dst_cat(inner)
        nat_trans_component_src(inner, inner.src_functor.src_cat.src(m))
        // we don't need src for inner here; need src_cat.src(K(m)) = src_cat.dst(β(ms))
        nat_trans_component_dst(inner, inner.src_functor.src_cat.src(m))
        functor_src(inner.dst_functor, m)
        functor_compose(outer.src_functor,
            inner.dst_functor.mor_map(m),
            inner.component(inner.src_functor.src_cat.src(m)))
        outer.src_functor.mor_map(outer.src_functor.src_cat.compose(
            inner.dst_functor.mor_map(m),
            inner.component(inner.src_functor.src_cat.src(m)))) = outer.src_functor.dst_cat.compose(
                outer.src_functor.mor_map(inner.dst_functor.mor_map(m)),
                outer.src_functor.mor_map(inner.component(inner.src_functor.src_cat.src(m))))
        outer.src_functor.dst_cat.compose(
            outer.src_functor.mor_map(inner.component(inner.src_functor.src_cat.dst(m))),
            outer.src_functor.mor_map(inner.src_functor.mor_map(m))) = outer.src_functor.dst_cat.compose(
                outer.src_functor.mor_map(inner.dst_functor.mor_map(m)),
                outer.src_functor.mor_map(inner.component(inner.src_functor.src_cat.src(m))))
    }
}

/// Outer naturality at `inner.dst_functor.mor_map(m)`, bridged to use inner.src_functor.src_cat sources.
theorem hc_outer_nat_at_K[O1, M1, O2, M2, O3, M3](
    outer: NaturalTransformation[O2, M2, O3, M3],
    inner: NaturalTransformation[O1, M1, O2, M2],
    m: M1
) {
    horizontal_composable(outer, inner) implies
        outer.src_functor.dst_cat.compose(
            outer.component(inner.dst_functor.obj_map(inner.src_functor.src_cat.dst(m))),
            outer.src_functor.mor_map(inner.dst_functor.mor_map(m))
        ) = outer.src_functor.dst_cat.compose(
            outer.dst_functor.mor_map(inner.dst_functor.mor_map(m)),
            outer.component(inner.dst_functor.obj_map(inner.src_functor.src_cat.src(m)))
        )
} by {
    if horizontal_composable(outer, inner) {
        nat_trans_naturality(outer, inner.dst_functor.mor_map(m))
        outer.src_functor.src_cat.src = inner.src_functor.dst_cat.src
        outer.src_functor.src_cat.dst = inner.src_functor.dst_cat.dst
        nat_trans_shared_src_cat(inner)
        nat_trans_shared_dst_cat(inner)
        functor_dst(inner.dst_functor, m)
        functor_src(inner.dst_functor, m)
        outer.src_functor.dst_cat.compose(outer.component(inner.dst_functor.obj_map(inner.src_functor.src_cat.dst(m))), outer.src_functor.mor_map(inner.dst_functor.mor_map(m))) = outer.src_functor.dst_cat.compose(outer.dst_functor.mor_map(inner.dst_functor.mor_map(m)), outer.component(inner.dst_functor.obj_map(inner.src_functor.src_cat.src(m))))
    }
}

/// Composability needed for assoc step 1: α(K(md)) ∘ F(β(md)) is composable with F(H(m)).
theorem hc_assoc1_composable[O1, M1, O2, M2, O3, M3](
    outer: NaturalTransformation[O2, M2, O3, M3],
    inner: NaturalTransformation[O1, M1, O2, M2],
    m: M1
) {
    horizontal_composable(outer, inner) implies
        outer.src_functor.dst_cat.src(outer.src_functor.mor_map(inner.component(inner.src_functor.src_cat.dst(m))))
            = outer.src_functor.dst_cat.dst(outer.src_functor.mor_map(inner.src_functor.mor_map(m)))
} by {
    if horizontal_composable(outer, inner) {
        hc_F_beta_F_H_composable(outer, inner, m)
    }
}

/// Composability: α(K(md)) is composable with F(K(m)).
theorem hc_alpha_F_K_composable[O1, M1, O2, M2, O3, M3](
    outer: NaturalTransformation[O2, M2, O3, M3],
    inner: NaturalTransformation[O1, M1, O2, M2],
    m: M1
) {
    horizontal_composable(outer, inner) implies
        outer.src_functor.dst_cat.src(outer.component(inner.dst_functor.obj_map(inner.src_functor.src_cat.dst(m))))
            = outer.src_functor.dst_cat.dst(outer.src_functor.mor_map(inner.dst_functor.mor_map(m)))
} by {
    if horizontal_composable(outer, inner) {
        nat_trans_component_src(outer, inner.dst_functor.obj_map(inner.src_functor.src_cat.dst(m)))
        functor_dst(outer.src_functor, inner.dst_functor.mor_map(m))
        outer.src_functor.src_cat.dst = inner.src_functor.dst_cat.dst
        nat_trans_shared_src_cat(inner)
        functor_dst(inner.dst_functor, m)
        outer.src_functor.dst_cat.dst(outer.src_functor.mor_map(inner.dst_functor.mor_map(m))) = outer.src_functor.obj_map(inner.dst_functor.obj_map(inner.src_functor.src_cat.dst(m)))
    }
}

/// Composability: G(K(m)) is composable with α(K(ms)).
theorem hc_G_K_alpha_composable[O1, M1, O2, M2, O3, M3](
    outer: NaturalTransformation[O2, M2, O3, M3],
    inner: NaturalTransformation[O1, M1, O2, M2],
    m: M1
) {
    horizontal_composable(outer, inner) implies
        outer.src_functor.dst_cat.src(outer.dst_functor.mor_map(inner.dst_functor.mor_map(m)))
            = outer.src_functor.dst_cat.dst(outer.component(inner.dst_functor.obj_map(inner.src_functor.src_cat.src(m))))
} by {
    if horizontal_composable(outer, inner) {
        nat_trans_shared_dst_cat(outer)
        functor_src(outer.dst_functor, inner.dst_functor.mor_map(m))
        nat_trans_shared_src_cat(outer)
        outer.dst_functor.src_cat.src = inner.src_functor.dst_cat.src
        nat_trans_shared_src_cat(inner)
        functor_src(inner.dst_functor, m)
        nat_trans_shared_dst_cat(inner)
        nat_trans_component_dst(outer, inner.dst_functor.obj_map(inner.src_functor.src_cat.src(m)))
        outer.src_functor.dst_cat.src(outer.dst_functor.mor_map(inner.dst_functor.mor_map(m))) = outer.src_functor.dst_cat.dst(outer.component(inner.dst_functor.obj_map(inner.src_functor.src_cat.src(m))))
    }
}

/// Naturality of the horizontal composition component, expressed without the bundled composite functors.
theorem hc_naturality_pure[O1, M1, O2, M2, O3, M3](
    outer: NaturalTransformation[O2, M2, O3, M3],
    inner: NaturalTransformation[O1, M1, O2, M2],
    m: M1
) {
    horizontal_composable(outer, inner) implies
        outer.src_functor.dst_cat.compose(
            horizontal_compose_component(outer, inner, inner.src_functor.src_cat.dst(m)),
            outer.src_functor.mor_map(inner.src_functor.mor_map(m))
        ) = outer.src_functor.dst_cat.compose(
            outer.dst_functor.mor_map(inner.dst_functor.mor_map(m)),
            horizontal_compose_component(outer, inner, inner.src_functor.src_cat.src(m))
        )
} by {
    if horizontal_composable(outer, inner) {
        // Unfold component definitions.
        horizontal_compose_component(outer, inner, inner.src_functor.src_cat.dst(m)) = outer.src_functor.dst_cat.compose(
                outer.component(inner.dst_functor.obj_map(inner.src_functor.src_cat.dst(m))),
                outer.src_functor.mor_map(inner.component(inner.src_functor.src_cat.dst(m))))
        // Step 1: assoc on LHS — pull α(K(md)) outside.
        horizontal_compose_components_composable(outer, inner, inner.src_functor.src_cat.dst(m))
        hc_assoc1_composable(outer, inner, m)
        category_compose_assoc(outer.src_functor.dst_cat,
            outer.component(inner.dst_functor.obj_map(inner.src_functor.src_cat.dst(m))),
            outer.src_functor.mor_map(inner.component(inner.src_functor.src_cat.dst(m))),
            outer.src_functor.mor_map(inner.src_functor.mor_map(m)))
        // Step 2: rewrite F(β(md)) ∘ F(H(m)) into F(K(m)) ∘ F(β(ms)).
        hc_inner_nat_lifted(outer, inner, m)
        // Step 3: assoc back — pull F(β(ms)) out.
        hc_alpha_F_K_composable(outer, inner, m)
        hc_F_K_F_beta_composable(outer, inner, m)
        category_compose_assoc(outer.src_functor.dst_cat,
            outer.component(inner.dst_functor.obj_map(inner.src_functor.src_cat.dst(m))),
            outer.src_functor.mor_map(inner.dst_functor.mor_map(m)),
            outer.src_functor.mor_map(inner.component(inner.src_functor.src_cat.src(m))))
        outer.src_functor.dst_cat.compose(
            outer.component(inner.dst_functor.obj_map(inner.src_functor.src_cat.dst(m))),
            outer.src_functor.dst_cat.compose(
                outer.src_functor.mor_map(inner.dst_functor.mor_map(m)),
                outer.src_functor.mor_map(inner.component(inner.src_functor.src_cat.src(m))))) = outer.src_functor.dst_cat.compose(
                outer.src_functor.dst_cat.compose(
                    outer.component(inner.dst_functor.obj_map(inner.src_functor.src_cat.dst(m))),
                    outer.src_functor.mor_map(inner.dst_functor.mor_map(m))),
                outer.src_functor.mor_map(inner.component(inner.src_functor.src_cat.src(m))))
        // Step 4: outer naturality at K(m).
        hc_outer_nat_at_K(outer, inner, m)
        // Step 5: assoc — pull G(K(m)) outside.
        hc_G_K_alpha_composable(outer, inner, m)
        horizontal_compose_components_composable(outer, inner, inner.src_functor.src_cat.src(m))
        category_compose_assoc(outer.src_functor.dst_cat,
            outer.dst_functor.mor_map(inner.dst_functor.mor_map(m)),
            outer.component(inner.dst_functor.obj_map(inner.src_functor.src_cat.src(m))),
            outer.src_functor.mor_map(inner.component(inner.src_functor.src_cat.src(m))))
        // Explicit chain combining steps 0–5 above.
        outer.src_functor.dst_cat.compose(
            horizontal_compose_component(outer, inner, inner.src_functor.src_cat.dst(m)),
            outer.src_functor.mor_map(inner.src_functor.mor_map(m))) = outer.src_functor.dst_cat.compose(
                outer.component(inner.dst_functor.obj_map(inner.src_functor.src_cat.dst(m))),
                outer.src_functor.dst_cat.compose(
                    outer.src_functor.mor_map(inner.component(inner.src_functor.src_cat.dst(m))),
                    outer.src_functor.mor_map(inner.src_functor.mor_map(m))))
        outer.src_functor.dst_cat.compose(
            horizontal_compose_component(outer, inner, inner.src_functor.src_cat.dst(m)),
            outer.src_functor.mor_map(inner.src_functor.mor_map(m))) = outer.src_functor.dst_cat.compose(
                outer.src_functor.dst_cat.compose(
                    outer.dst_functor.mor_map(inner.dst_functor.mor_map(m)),
                    outer.component(inner.dst_functor.obj_map(inner.src_functor.src_cat.src(m)))),
                outer.src_functor.mor_map(inner.component(inner.src_functor.src_cat.src(m))))
        outer.src_functor.dst_cat.compose(
            horizontal_compose_component(outer, inner, inner.src_functor.src_cat.dst(m)),
            outer.src_functor.mor_map(inner.src_functor.mor_map(m))) = outer.src_functor.dst_cat.compose(
                outer.dst_functor.mor_map(inner.dst_functor.mor_map(m)),
                outer.src_functor.dst_cat.compose(
                    outer.component(inner.dst_functor.obj_map(inner.src_functor.src_cat.src(m))),
                    outer.src_functor.mor_map(inner.component(inner.src_functor.src_cat.src(m)))))
        outer.src_functor.dst_cat.compose(
            horizontal_compose_component(outer, inner, inner.src_functor.src_cat.dst(m)),
            outer.src_functor.mor_map(inner.src_functor.mor_map(m))) = outer.src_functor.dst_cat.compose(
                outer.dst_functor.mor_map(inner.dst_functor.mor_map(m)),
                horizontal_compose_component(outer, inner, inner.src_functor.src_cat.src(m)))
    }
}

/// The horizontal compose component map satisfies the naturality axiom relative to bundled composite functors.
theorem hc_naturality_axiom[O1, M1, O2, M2, O3, M3](
    outer: NaturalTransformation[O2, M2, O3, M3],
    inner: NaturalTransformation[O1, M1, O2, M2],
    src_comp: Functor[O1, M1, O3, M3],
    dst_comp: Functor[O1, M1, O3, M3]
) {
    horizontal_composable(outer, inner)
    and compose_functor(outer.src_functor, inner.src_functor) = Option.some(src_comp)
    and compose_functor(outer.dst_functor, inner.dst_functor) = Option.some(dst_comp)
    implies nat_trans_naturality_axiom(src_comp, dst_comp, horizontal_compose_component(outer, inner))
} by {
    if horizontal_composable(outer, inner)
    and compose_functor(outer.src_functor, inner.src_functor) = Option.some(src_comp)
    and compose_functor(outer.dst_functor, inner.dst_functor) = Option.some(dst_comp) {
        compose_functor_src_cat(outer.src_functor, inner.src_functor, src_comp)
        compose_functor_dst_cat(outer.src_functor, inner.src_functor, src_comp)
        compose_functor_mor_map(outer.src_functor, inner.src_functor, src_comp)
        compose_functor_mor_map(outer.dst_functor, inner.dst_functor, dst_comp)
        forall(m: M1) {
            src_comp.mor_map(m) = outer.src_functor.mor_map(inner.src_functor.mor_map(m))
            dst_comp.mor_map(m) = outer.dst_functor.mor_map(inner.dst_functor.mor_map(m))
            hc_naturality_pure(outer, inner, m)
            // Bridge src_cat sources/dsts: src_comp.src_cat = inner.src_functor.src_cat.
            src_comp.dst_cat.compose(
                horizontal_compose_component(outer, inner, src_comp.src_cat.dst(m)),
                src_comp.mor_map(m)) = src_comp.dst_cat.compose(
                    dst_comp.mor_map(m),
                    horizontal_compose_component(outer, inner, src_comp.src_cat.src(m)))
        }
    }
}

/// The horizontal compose component map is a natural transformation between bundled composite functors.
theorem hc_is_natural_transformation[O1, M1, O2, M2, O3, M3](
    outer: NaturalTransformation[O2, M2, O3, M3],
    inner: NaturalTransformation[O1, M1, O2, M2],
    src_comp: Functor[O1, M1, O3, M3],
    dst_comp: Functor[O1, M1, O3, M3]
) {
    horizontal_composable(outer, inner)
    and compose_functor(outer.src_functor, inner.src_functor) = Option.some(src_comp)
    and compose_functor(outer.dst_functor, inner.dst_functor) = Option.some(dst_comp)
    implies is_natural_transformation(src_comp, dst_comp, horizontal_compose_component(outer, inner))
} by {
    if horizontal_composable(outer, inner)
    and compose_functor(outer.src_functor, inner.src_functor) = Option.some(src_comp)
    and compose_functor(outer.dst_functor, inner.dst_functor) = Option.some(dst_comp) {
        compose_functor_src_cat(outer.src_functor, inner.src_functor, src_comp)
        compose_functor_dst_cat(outer.src_functor, inner.src_functor, src_comp)
        compose_functor_src_cat(outer.dst_functor, inner.dst_functor, dst_comp)
        compose_functor_dst_cat(outer.dst_functor, inner.dst_functor, dst_comp)
        nat_trans_shared_src_cat(inner)
        nat_trans_shared_dst_cat(outer)
        src_comp.dst_cat = dst_comp.dst_cat
        functors_parallel(src_comp, dst_comp)
        forall(x: O1) {
            horizontal_compose_component_src(outer, inner, src_comp, x)
            horizontal_compose_component_dst(outer, inner, dst_comp, x)
            src_comp.dst_cat.dst(horizontal_compose_component(outer, inner, x)) = dst_comp.obj_map(x)
            (src_comp.dst_cat.src(horizontal_compose_component(outer, inner, x)) = src_comp.obj_map(x) and src_comp.dst_cat.dst(horizontal_compose_component(outer, inner, x)) = dst_comp.obj_map(x))
        }
        nat_trans_endpoints_axiom(src_comp, dst_comp, horizontal_compose_component(outer, inner))
        hc_naturality_axiom(outer, inner, src_comp, dst_comp)
        is_natural_transformation(src_comp, dst_comp, horizontal_compose_component(outer, inner))
    }
}

/// The horizontal composition of two horizontally composable natural transformations is some bundled transformation.
theorem horizontal_compose_nat_trans_some[O1, M1, O2, M2, O3, M3](
    outer: NaturalTransformation[O2, M2, O3, M3],
    inner: NaturalTransformation[O1, M1, O2, M2],
    src_comp: Functor[O1, M1, O3, M3],
    dst_comp: Functor[O1, M1, O3, M3]
) {
    horizontal_composable(outer, inner)
    and compose_functor(outer.src_functor, inner.src_functor) = Option.some(src_comp)
    and compose_functor(outer.dst_functor, inner.dst_functor) = Option.some(dst_comp)
    implies exists(n: NaturalTransformation[O1, M1, O3, M3]) {
        horizontal_compose_nat_trans(outer, inner, src_comp, dst_comp) = Option.some(n)
    }
} by {
    if horizontal_composable(outer, inner)
    and compose_functor(outer.src_functor, inner.src_functor) = Option.some(src_comp)
    and compose_functor(outer.dst_functor, inner.dst_functor) = Option.some(dst_comp) {
        hc_is_natural_transformation(outer, inner, src_comp, dst_comp)
        let n: NaturalTransformation[O1, M1, O3, M3] satisfy {
            NaturalTransformation[O1, M1, O3, M3].new(
                src_comp, dst_comp, horizontal_compose_component(outer, inner)) = Option.some(n)
        }
        horizontal_compose_nat_trans(outer, inner, src_comp, dst_comp) = Option.some(n)
    }
}

/// Two natural transformations with equal source functor, target functor, and component map are equal.
theorem nat_trans_ext[O1, M1, O2, M2](
    a: NaturalTransformation[O1, M1, O2, M2], b: NaturalTransformation[O1, M1, O2, M2]
) {
    a.src_functor = b.src_functor
    and a.dst_functor = b.dst_functor
    and a.component = b.component
    implies a = b
}

/// At each object, vertical composition of the identity natural transformation on the left with a natural transformation reduces to its component.
theorem vertical_compose_id_left_component_at[O1, M1, O2, M2](
    a: NaturalTransformation[O1, M1, O2, M2], x: O1
) {
    vertical_compose_component(identity_nat_trans(a.dst_functor), a, x) = a.component(x)
} by {
    let id_a = identity_nat_trans(a.dst_functor)
    identity_nat_trans_src_functor(a.dst_functor)
    identity_nat_trans_dst_functor(a.dst_functor)
    identity_nat_trans_component_eq(a.dst_functor)
    identity_nat_trans_component(a.dst_functor)(x) = identity_nat_trans_component(a.dst_functor, x)
    nat_trans_shared_dst_cat(a)
    vertical_compose_component(id_a, a, x) = a.dst_functor.dst_cat.compose(a.dst_functor.dst_cat.identity(a.dst_functor.obj_map(x)), a.component(x))
    nat_trans_component_dst(a, x)
    category_id_left(a.dst_functor.dst_cat, a.component(x))
}

/// Vertical composition of the identity natural transformation on the left with a natural transformation reduces to it.
theorem vertical_compose_id_left[O1, M1, O2, M2](
    a: NaturalTransformation[O1, M1, O2, M2]
) {
    vertical_compose_nat_trans(identity_nat_trans(a.dst_functor), a) = Option.some(a)
} by {
    let id_a = identity_nat_trans(a.dst_functor)
    identity_nat_trans_src_functor(a.dst_functor)
    identity_nat_trans_dst_functor(a.dst_functor)
    vertical_compose_nat_trans_some(id_a, a)
    let n: NaturalTransformation[O1, M1, O2, M2] satisfy {
        vertical_compose_nat_trans(id_a, a) = Option.some(n)
    }
    vertical_compose_nat_trans_src_functor(id_a, a, n)
    vertical_compose_nat_trans_dst_functor(id_a, a, n)
    vertical_compose_nat_trans_component(id_a, a, n)
    forall(x: O1) {
        vertical_compose_id_left_component_at(a, x)
        vertical_compose_component(id_a, a)(x) = vertical_compose_component(id_a, a, x)
        n.component(x) = a.component(x)
    }
    nat_trans_ext(n, a)
}

/// At each object, vertical composition of a natural transformation with the identity natural transformation on the right reduces to its component.
theorem vertical_compose_id_right_component_at[O1, M1, O2, M2](
    a: NaturalTransformation[O1, M1, O2, M2], x: O1
) {
    vertical_compose_component(a, identity_nat_trans(a.src_functor), x) = a.component(x)
} by {
    let id_a = identity_nat_trans(a.src_functor)
    identity_nat_trans_src_functor(a.src_functor)
    identity_nat_trans_dst_functor(a.src_functor)
    identity_nat_trans_component_eq(a.src_functor)
    identity_nat_trans_component(a.src_functor)(x) = identity_nat_trans_component(a.src_functor, x)
    vertical_compose_component(a, id_a, x) = a.src_functor.dst_cat.compose(a.component(x), a.src_functor.dst_cat.identity(a.src_functor.obj_map(x)))
    nat_trans_component_src(a, x)
    category_id_right(a.src_functor.dst_cat, a.component(x))
}

/// Vertical composition of a natural transformation with the identity natural transformation on the right reduces to it.
theorem vertical_compose_id_right[O1, M1, O2, M2](
    a: NaturalTransformation[O1, M1, O2, M2]
) {
    vertical_compose_nat_trans(a, identity_nat_trans(a.src_functor)) = Option.some(a)
} by {
    let id_a = identity_nat_trans(a.src_functor)
    identity_nat_trans_src_functor(a.src_functor)
    identity_nat_trans_dst_functor(a.src_functor)
    vertical_compose_nat_trans_some(a, id_a)
    let n: NaturalTransformation[O1, M1, O2, M2] satisfy {
        vertical_compose_nat_trans(a, id_a) = Option.some(n)
    }
    vertical_compose_nat_trans_src_functor(a, id_a, n)
    vertical_compose_nat_trans_dst_functor(a, id_a, n)
    vertical_compose_nat_trans_component(a, id_a, n)
    forall(x: O1) {
        vertical_compose_id_right_component_at(a, x)
        vertical_compose_component(a, id_a)(x) = vertical_compose_component(a, id_a, x)
        n.component(x) = a.component(x)
    }
    nat_trans_ext(n, a)
}
