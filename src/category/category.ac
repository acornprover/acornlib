/// True if the source and target of every identity morphism are the underlying object.
define category_id_endpoints_constraint[O, M](
    src: M -> O,
    dst: M -> O,
    identity: O -> M
) -> Bool {
    forall(x: O) {
        src(identity(x)) = x and dst(identity(x)) = x
    }
}

/// True if the source of every composable composition is the source of the inner morphism.
define category_compose_src_constraint[O, M](
    src: M -> O,
    dst: M -> O,
    compose: (M, M) -> M
) -> Bool {
    forall(f: M, g: M) {
        src(f) = dst(g) implies src(compose(f, g)) = src(g)
    }
}

/// True if the target of every composable composition is the target of the outer morphism.
define category_compose_dst_constraint[O, M](
    src: M -> O,
    dst: M -> O,
    compose: (M, M) -> M
) -> Bool {
    forall(f: M, g: M) {
        src(f) = dst(g) implies dst(compose(f, g)) = dst(f)
    }
}

/// True if composition is associative on every composable triple of morphisms.
define category_assoc_constraint[O, M](
    src: M -> O,
    dst: M -> O,
    compose: (M, M) -> M
) -> Bool {
    forall(f: M, g: M, h: M) {
        src(f) = dst(g) and src(g) = dst(h) implies
        compose(compose(f, g), h) = compose(f, compose(g, h))
    }
}

/// True if the identity morphism on the target acts as a left unit for composition.
define category_id_left_constraint[O, M](
    src: M -> O,
    dst: M -> O,
    identity: O -> M,
    compose: (M, M) -> M
) -> Bool {
    forall(f: M) {
        compose(identity(dst(f)), f) = f
    }
}

/// True if the identity morphism on the source acts as a right unit for composition.
define category_id_right_constraint[O, M](
    src: M -> O,
    dst: M -> O,
    identity: O -> M,
    compose: (M, M) -> M
) -> Bool {
    forall(f: M) {
        compose(f, identity(src(f))) = f
    }
}

/// True if the given source, target, identity and composition data form a category.
define is_category[O, M](
    src: M -> O,
    dst: M -> O,
    identity: O -> M,
    compose: (M, M) -> M
) -> Bool {
    category_id_endpoints_constraint(src, dst, identity)
    and category_compose_src_constraint(src, dst, compose)
    and category_compose_dst_constraint(src, dst, compose)
    and category_assoc_constraint(src, dst, compose)
    and category_id_left_constraint(src, dst, identity, compose)
    and category_id_right_constraint(src, dst, identity, compose)
}

/// A category with object type `O` and morphism type `M`.
/// The composition operation is total; its behavior on non-composable pairs is unconstrained.
structure Category[O, M] {
    /// The source object of a morphism.
    src: M -> O

    /// The target object of a morphism.
    dst: M -> O

    /// The identity morphism on each object.
    identity: O -> M

    /// Composition of morphisms; `compose(f, g)` is `f` after `g`, valid when `src(f) = dst(g)`.
    compose: (M, M) -> M
} constraint {
    is_category(src, dst, identity, compose)
}

/// Category extensionality from pointwise equality of every component.
theorem category_ext[O, M](c: Category[O, M], d: Category[O, M]) {
    (forall(m: M) { c.src(m) = d.src(m) })
    and (forall(m: M) { c.dst(m) = d.dst(m) })
    and (forall(x: O) { c.identity(x) = d.identity(x) })
    and (forall(f: M, g: M) { c.compose(f, g) = d.compose(f, g) })
    implies c = d
} by {
    if (forall(m: M) { c.src(m) = d.src(m) })
        and (forall(m: M) { c.dst(m) = d.dst(m) })
        and (forall(x: O) { c.identity(x) = d.identity(x) })
        and (forall(f: M, g: M) { c.compose(f, g) = d.compose(f, g) }) {
        forall(f: M) {
            forall(g: M) {
                c.compose(f, g) = d.compose(f, g)
            }
            c.compose(f) = d.compose(f)
        }
        c.compose = d.compose
    }
}

/// Construction of a category remembers its source map.
theorem category_new_src[O, M](
    src: M -> O,
    dst: M -> O,
    identity: O -> M,
    compose: (M, M) -> M,
    c: Category[O, M]
) {
    Category[O, M].new(src, dst, identity, compose) = Option.some(c) implies c.src = src
}

/// Construction of a category remembers its target map.
theorem category_new_dst[O, M](
    src: M -> O,
    dst: M -> O,
    identity: O -> M,
    compose: (M, M) -> M,
    c: Category[O, M]
) {
    Category[O, M].new(src, dst, identity, compose) = Option.some(c) implies c.dst = dst
}

/// Construction of a category remembers its identity-assigning map.
theorem category_new_identity[O, M](
    src: M -> O,
    dst: M -> O,
    identity: O -> M,
    compose: (M, M) -> M,
    c: Category[O, M]
) {
    Category[O, M].new(src, dst, identity, compose) = Option.some(c) implies c.identity = identity
}

/// Construction of a category remembers its composition operator.
theorem category_new_compose[O, M](
    src: M -> O,
    dst: M -> O,
    identity: O -> M,
    compose: (M, M) -> M,
    c: Category[O, M]
) {
    Category[O, M].new(src, dst, identity, compose) = Option.some(c) implies c.compose = compose
}

/// The source of an identity morphism is its underlying object.
theorem category_identity_src[O, M](c: Category[O, M], x: O) {
    c.src(c.identity(x)) = x
} by {
    is_category(c.src, c.dst, c.identity, c.compose) =
        (category_id_endpoints_constraint(c.src, c.dst, c.identity)
         and category_compose_src_constraint(c.src, c.dst, c.compose)
         and category_compose_dst_constraint(c.src, c.dst, c.compose)
         and category_assoc_constraint(c.src, c.dst, c.compose)
         and category_id_left_constraint(c.src, c.dst, c.identity, c.compose)
         and category_id_right_constraint(c.src, c.dst, c.identity, c.compose))
    category_id_endpoints_constraint(c.src, c.dst, c.identity) = forall(y: O) {
        c.src(c.identity(y)) = y and c.dst(c.identity(y)) = y
    }
}

/// The target of an identity morphism is its underlying object.
theorem category_identity_dst[O, M](c: Category[O, M], x: O) {
    c.dst(c.identity(x)) = x
} by {
    is_category(c.src, c.dst, c.identity, c.compose) =
        (category_id_endpoints_constraint(c.src, c.dst, c.identity)
         and category_compose_src_constraint(c.src, c.dst, c.compose)
         and category_compose_dst_constraint(c.src, c.dst, c.compose)
         and category_assoc_constraint(c.src, c.dst, c.compose)
         and category_id_left_constraint(c.src, c.dst, c.identity, c.compose)
         and category_id_right_constraint(c.src, c.dst, c.identity, c.compose))
    category_id_endpoints_constraint(c.src, c.dst, c.identity) = forall(y: O) {
        c.src(c.identity(y)) = y and c.dst(c.identity(y)) = y
    }
}

/// The source of a composition is the source of the inner morphism.
theorem category_compose_src[O, M](c: Category[O, M], f: M, g: M) {
    c.src(f) = c.dst(g) implies c.src(c.compose(f, g)) = c.src(g)
} by {
    is_category(c.src, c.dst, c.identity, c.compose) =
        (category_id_endpoints_constraint(c.src, c.dst, c.identity)
         and category_compose_src_constraint(c.src, c.dst, c.compose)
         and category_compose_dst_constraint(c.src, c.dst, c.compose)
         and category_assoc_constraint(c.src, c.dst, c.compose)
         and category_id_left_constraint(c.src, c.dst, c.identity, c.compose)
         and category_id_right_constraint(c.src, c.dst, c.identity, c.compose))
    category_compose_src_constraint(c.src, c.dst, c.compose) = forall(p: M, q: M) {
        c.src(p) = c.dst(q) implies c.src(c.compose(p, q)) = c.src(q)
    }
}

/// The target of a composition is the target of the outer morphism.
theorem category_compose_dst[O, M](c: Category[O, M], f: M, g: M) {
    c.src(f) = c.dst(g) implies c.dst(c.compose(f, g)) = c.dst(f)
} by {
    is_category(c.src, c.dst, c.identity, c.compose) =
        (category_id_endpoints_constraint(c.src, c.dst, c.identity)
         and category_compose_src_constraint(c.src, c.dst, c.compose)
         and category_compose_dst_constraint(c.src, c.dst, c.compose)
         and category_assoc_constraint(c.src, c.dst, c.compose)
         and category_id_left_constraint(c.src, c.dst, c.identity, c.compose)
         and category_id_right_constraint(c.src, c.dst, c.identity, c.compose))
    category_compose_dst_constraint(c.src, c.dst, c.compose) = forall(p: M, q: M) {
        c.src(p) = c.dst(q) implies c.dst(c.compose(p, q)) = c.dst(p)
    }
}

/// Composition is associative on every composable triple of morphisms.
theorem category_compose_assoc[O, M](c: Category[O, M], f: M, g: M, h: M) {
    c.src(f) = c.dst(g) and c.src(g) = c.dst(h) implies
    c.compose(c.compose(f, g), h) = c.compose(f, c.compose(g, h))
} by {
    is_category(c.src, c.dst, c.identity, c.compose) =
        (category_id_endpoints_constraint(c.src, c.dst, c.identity)
         and category_compose_src_constraint(c.src, c.dst, c.compose)
         and category_compose_dst_constraint(c.src, c.dst, c.compose)
         and category_assoc_constraint(c.src, c.dst, c.compose)
         and category_id_left_constraint(c.src, c.dst, c.identity, c.compose)
         and category_id_right_constraint(c.src, c.dst, c.identity, c.compose))
    category_assoc_constraint(c.src, c.dst, c.compose) =
        forall(p: M, q: M, r: M) {
            c.src(p) = c.dst(q) and c.src(q) = c.dst(r) implies
            c.compose(c.compose(p, q), r) = c.compose(p, c.compose(q, r))
        }
}

/// The identity morphism on the target acts as a left unit for composition.
theorem category_id_left[O, M](c: Category[O, M], f: M) {
    c.compose(c.identity(c.dst(f)), f) = f
} by {
    is_category(c.src, c.dst, c.identity, c.compose) =
        (category_id_endpoints_constraint(c.src, c.dst, c.identity)
         and category_compose_src_constraint(c.src, c.dst, c.compose)
         and category_compose_dst_constraint(c.src, c.dst, c.compose)
         and category_assoc_constraint(c.src, c.dst, c.compose)
         and category_id_left_constraint(c.src, c.dst, c.identity, c.compose)
         and category_id_right_constraint(c.src, c.dst, c.identity, c.compose))
    category_id_left_constraint(c.src, c.dst, c.identity, c.compose) = forall(p: M) {
        c.compose(c.identity(c.dst(p)), p) = p
    }
    forall(p: M) {
        c.compose(c.identity(c.dst(p)), p) = p
    }
}

/// The identity morphism on the source acts as a right unit for composition.
theorem category_id_right[O, M](c: Category[O, M], f: M) {
    c.compose(f, c.identity(c.src(f))) = f
} by {
    is_category(c.src, c.dst, c.identity, c.compose)
    forall(p: M) {
        c.compose(p, c.identity(c.src(p))) = p
    }
}

/// True if a pair of morphisms `(f, g)` is composable, i.e. the source of `f` matches the target of `g`.
define composable[O, M](c: Category[O, M], f: M, g: M) -> Bool {
    c.src(f) = c.dst(g)
}

/// Identity morphisms are composable on the right with any morphism whose target equals the underlying object.
theorem composable_identity_left[O, M](c: Category[O, M], f: M) {
    composable(c, c.identity(c.dst(f)), f)
} by {
    category_identity_src(c, c.dst(f))
}

/// Identity morphisms are composable on the left with any morphism whose source equals the underlying object.
theorem composable_identity_right[O, M](c: Category[O, M], f: M) {
    composable(c, f, c.identity(c.src(f)))
} by {
    category_identity_dst(c, c.src(f))
}

/// Composing two composable pairs of morphisms produces another composable pair.
theorem composable_compose_left[O, M](c: Category[O, M], f: M, g: M, h: M) {
    composable(c, f, g) and composable(c, g, h) implies
    composable(c, c.compose(f, g), h)
} by {
    if composable(c, f, g) and composable(c, g, h) {
        category_compose_src(c, f, g)
        c.src(c.compose(f, g)) = c.src(g)
        c.src(c.compose(f, g)) = c.dst(h)
    }
}

/// Composing on the right preserves composability with an outer morphism.
theorem composable_compose_right[O, M](c: Category[O, M], f: M, g: M, h: M) {
    composable(c, f, g) and composable(c, g, h) implies
    composable(c, f, c.compose(g, h))
} by {
    if composable(c, f, g) and composable(c, g, h) {
        category_compose_dst(c, g, h)
        c.src(f) = c.dst(c.compose(g, h))
    }
}
