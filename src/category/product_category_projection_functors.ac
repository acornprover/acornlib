/// Projection functors out of a binary product category.

from category.category import Category
from category.functor import Functor, is_functor, functor_src_axiom, functor_dst_axiom,
    functor_identity_axiom, functor_compose_axiom, functor_new_src_cat,
    functor_new_dst_cat, functor_new_obj_map, functor_new_mor_map
from pair import Pair
from category.product_category import product_category, product_src, product_dst,
    product_identity, product_compose, product_src_first, product_src_second,
    product_dst_first, product_dst_second, product_identity_first,
    product_identity_second, product_compose_first, product_compose_second,
    product_category_src, product_category_dst, product_category_identity,
    product_category_compose

/// The object map of the first projection functor.
define product_fst_obj[O1, O2](x: Pair[O1, O2]) -> O1 {
    x.first
}

/// The object map of the second projection functor.
define product_snd_obj[O1, O2](x: Pair[O1, O2]) -> O2 {
    x.second
}

/// The morphism map of the first projection functor.
define product_fst_mor[M1, M2](f: Pair[M1, M2]) -> M1 {
    f.first
}

/// The morphism map of the second projection functor.
define product_snd_mor[M1, M2](f: Pair[M1, M2]) -> M2 {
    f.second
}

/// The first projections of objects and morphisms form a functor from the product category.
theorem product_fst_is_functor[O1, M1, O2, M2](
    c: Category[O1, M1], d: Category[O2, M2]
) {
    is_functor(product_category(c, d), c, product_fst_obj[O1, O2], product_fst_mor[M1, M2])
} by {
    let p = product_category(c, d)
    product_category_src(c, d)
    product_category_dst(c, d)
    product_category_identity(c, d)
    product_category_compose(c, d)
    forall(f: Pair[M1, M2]) {
        product_fst_mor[M1, M2](f) = f.first
        p.src = product_src(c, d)
        p.src(f) = product_src(c, d, f)
        product_src_first(c, d, f)
        product_src(c, d, f).first = c.src(f.first)
        p.src(f).first = c.src(f.first)
        product_fst_obj[O1, O2](p.src(f)) = p.src(f).first
        product_fst_obj[O1, O2](p.src(f)) = c.src(f.first)
        c.src(product_fst_mor[M1, M2](f)) = c.src(f.first)
        c.src(product_fst_mor[M1, M2](f)) = product_fst_obj[O1, O2](p.src(f))
    }
    functor_src_axiom(p, c, product_fst_obj[O1, O2], product_fst_mor[M1, M2])
    forall(f: Pair[M1, M2]) {
        product_fst_mor[M1, M2](f) = f.first
        p.dst = product_dst(c, d)
        p.dst(f) = product_dst(c, d, f)
        product_dst_first(c, d, f)
        product_dst(c, d, f).first = c.dst(f.first)
        p.dst(f).first = c.dst(f.first)
        product_fst_obj[O1, O2](p.dst(f)) = p.dst(f).first
        product_fst_obj[O1, O2](p.dst(f)) = c.dst(f.first)
        c.dst(product_fst_mor[M1, M2](f)) = c.dst(f.first)
        c.dst(product_fst_mor[M1, M2](f)) = product_fst_obj[O1, O2](p.dst(f))
    }
    functor_dst_axiom(p, c, product_fst_obj[O1, O2], product_fst_mor[M1, M2])
    forall(x: Pair[O1, O2]) {
        p.identity = product_identity(c, d)
        p.identity(x) = product_identity(c, d, x)
        product_identity_first(c, d, x)
        product_identity(c, d, x).first = c.identity(x.first)
        product_fst_mor[M1, M2](p.identity(x)) = p.identity(x).first
        product_fst_mor[M1, M2](p.identity(x)) = c.identity(x.first)
        product_fst_obj[O1, O2](x) = x.first
        c.identity(product_fst_obj[O1, O2](x)) = c.identity(x.first)
        product_fst_mor[M1, M2](p.identity(x)) = c.identity(product_fst_obj[O1, O2](x))
    }
    functor_identity_axiom(p, c, product_fst_obj[O1, O2], product_fst_mor[M1, M2])
    forall(f: Pair[M1, M2], g: Pair[M1, M2]) {
        if p.src(f) = p.dst(g) {
            p.compose = product_compose(c, d)
            p.compose(f, g) = product_compose(c, d, f, g)
            product_compose_first(c, d, f, g)
            product_compose(c, d, f, g).first = c.compose(f.first, g.first)
            p.compose(f, g).first = c.compose(f.first, g.first)
            product_fst_mor[M1, M2](p.compose(f, g)) = p.compose(f, g).first
            product_fst_mor[M1, M2](p.compose(f, g)) = c.compose(f.first, g.first)
            product_fst_mor[M1, M2](f) = f.first
            product_fst_mor[M1, M2](g) = g.first
            c.compose(product_fst_mor[M1, M2](f), product_fst_mor[M1, M2](g)) = c.compose(f.first, g.first)
            product_fst_mor[M1, M2](p.compose(f, g)) = c.compose(product_fst_mor[M1, M2](f), product_fst_mor[M1, M2](g))
        }
    }
    functor_compose_axiom(p, c, product_fst_obj[O1, O2], product_fst_mor[M1, M2])
    is_functor(p, c, product_fst_obj[O1, O2], product_fst_mor[M1, M2]) =
        (functor_src_axiom(p, c, product_fst_obj[O1, O2], product_fst_mor[M1, M2])
        and functor_dst_axiom(p, c, product_fst_obj[O1, O2], product_fst_mor[M1, M2])
        and functor_identity_axiom(p, c, product_fst_obj[O1, O2], product_fst_mor[M1, M2])
        and functor_compose_axiom(p, c, product_fst_obj[O1, O2], product_fst_mor[M1, M2]))
    is_functor(p, c, product_fst_obj[O1, O2], product_fst_mor[M1, M2])
}

/// The second projections of objects and morphisms form a functor from the product category.
theorem product_snd_is_functor[O1, M1, O2, M2](
    c: Category[O1, M1], d: Category[O2, M2]
) {
    is_functor(product_category(c, d), d, product_snd_obj[O1, O2], product_snd_mor[M1, M2])
} by {
    let p = product_category(c, d)
    product_category_src(c, d)
    product_category_dst(c, d)
    product_category_identity(c, d)
    product_category_compose(c, d)
    forall(f: Pair[M1, M2]) {
        product_snd_mor[M1, M2](f) = f.second
        p.src = product_src(c, d)
        p.src(f) = product_src(c, d, f)
        product_src_second(c, d, f)
        product_src(c, d, f).second = d.src(f.second)
        p.src(f).second = d.src(f.second)
        product_snd_obj[O1, O2](p.src(f)) = p.src(f).second
        product_snd_obj[O1, O2](p.src(f)) = d.src(f.second)
        d.src(product_snd_mor[M1, M2](f)) = d.src(f.second)
        d.src(product_snd_mor[M1, M2](f)) = product_snd_obj[O1, O2](p.src(f))
    }
    functor_src_axiom(p, d, product_snd_obj[O1, O2], product_snd_mor[M1, M2])
    forall(f: Pair[M1, M2]) {
        product_snd_mor[M1, M2](f) = f.second
        p.dst = product_dst(c, d)
        p.dst(f) = product_dst(c, d, f)
        product_dst_second(c, d, f)
        product_dst(c, d, f).second = d.dst(f.second)
        p.dst(f).second = d.dst(f.second)
        product_snd_obj[O1, O2](p.dst(f)) = p.dst(f).second
        product_snd_obj[O1, O2](p.dst(f)) = d.dst(f.second)
        d.dst(product_snd_mor[M1, M2](f)) = d.dst(f.second)
        d.dst(product_snd_mor[M1, M2](f)) = product_snd_obj[O1, O2](p.dst(f))
    }
    functor_dst_axiom(p, d, product_snd_obj[O1, O2], product_snd_mor[M1, M2])
    forall(x: Pair[O1, O2]) {
        p.identity = product_identity(c, d)
        p.identity(x) = product_identity(c, d, x)
        product_identity_second(c, d, x)
        product_identity(c, d, x).second = d.identity(x.second)
        product_snd_mor[M1, M2](p.identity(x)) = p.identity(x).second
        product_snd_mor[M1, M2](p.identity(x)) = d.identity(x.second)
        product_snd_obj[O1, O2](x) = x.second
        d.identity(product_snd_obj[O1, O2](x)) = d.identity(x.second)
        product_snd_mor[M1, M2](p.identity(x)) = d.identity(product_snd_obj[O1, O2](x))
    }
    functor_identity_axiom(p, d, product_snd_obj[O1, O2], product_snd_mor[M1, M2])
    forall(f: Pair[M1, M2], g: Pair[M1, M2]) {
        if p.src(f) = p.dst(g) {
            p.compose = product_compose(c, d)
            p.compose(f, g) = product_compose(c, d, f, g)
            product_compose_second(c, d, f, g)
            product_compose(c, d, f, g).second = d.compose(f.second, g.second)
            p.compose(f, g).second = d.compose(f.second, g.second)
            product_snd_mor[M1, M2](p.compose(f, g)) = p.compose(f, g).second
            product_snd_mor[M1, M2](p.compose(f, g)) = d.compose(f.second, g.second)
            product_snd_mor[M1, M2](f) = f.second
            product_snd_mor[M1, M2](g) = g.second
            d.compose(product_snd_mor[M1, M2](f), product_snd_mor[M1, M2](g)) = d.compose(f.second, g.second)
            product_snd_mor[M1, M2](p.compose(f, g)) = d.compose(product_snd_mor[M1, M2](f), product_snd_mor[M1, M2](g))
        }
    }
    functor_compose_axiom(p, d, product_snd_obj[O1, O2], product_snd_mor[M1, M2])
    is_functor(p, d, product_snd_obj[O1, O2], product_snd_mor[M1, M2]) =
        (functor_src_axiom(p, d, product_snd_obj[O1, O2], product_snd_mor[M1, M2])
        and functor_dst_axiom(p, d, product_snd_obj[O1, O2], product_snd_mor[M1, M2])
        and functor_identity_axiom(p, d, product_snd_obj[O1, O2], product_snd_mor[M1, M2])
        and functor_compose_axiom(p, d, product_snd_obj[O1, O2], product_snd_mor[M1, M2]))
    is_functor(p, d, product_snd_obj[O1, O2], product_snd_mor[M1, M2])
}

/// The first projection functor out of a binary product category.
let product_fst_functor[O1, M1, O2, M2](c: Category[O1, M1], d: Category[O2, M2]) -> result: Functor[Pair[O1, O2], Pair[M1, M2], O1, M1] satisfy {
    Functor[Pair[O1, O2], Pair[M1, M2], O1, M1].new(
        product_category(c, d), c, product_fst_obj[O1, O2], product_fst_mor[M1, M2]
    ) = Option.some(result)
} by {
    product_fst_is_functor(c, d)
}

/// The second projection functor out of a binary product category.
let product_snd_functor[O1, M1, O2, M2](c: Category[O1, M1], d: Category[O2, M2]) -> result: Functor[Pair[O1, O2], Pair[M1, M2], O2, M2] satisfy {
    Functor[Pair[O1, O2], Pair[M1, M2], O2, M2].new(
        product_category(c, d), d, product_snd_obj[O1, O2], product_snd_mor[M1, M2]
    ) = Option.some(result)
} by {
    product_snd_is_functor(c, d)
}

/// The first projection functor has the product category as source.
theorem product_fst_functor_src_cat[O1, M1, O2, M2](
    c: Category[O1, M1], d: Category[O2, M2]
) {
    product_fst_functor(c, d).src_cat = product_category(c, d)
} by {
    let f = product_fst_functor(c, d)
    (Functor[Pair[O1, O2], Pair[M1, M2], O1, M1].new(
        product_category(c, d), c, product_fst_obj[O1, O2], product_fst_mor[M1, M2]
    ) = Option.some(f))
    functor_new_src_cat(product_category(c, d), c, product_fst_obj[O1, O2], product_fst_mor[M1, M2], f)
}

/// The first projection functor has the first category as target.
theorem product_fst_functor_dst_cat[O1, M1, O2, M2](
    c: Category[O1, M1], d: Category[O2, M2]
) {
    product_fst_functor(c, d).dst_cat = c
} by {
    let f = product_fst_functor(c, d)
    (Functor[Pair[O1, O2], Pair[M1, M2], O1, M1].new(
        product_category(c, d), c, product_fst_obj[O1, O2], product_fst_mor[M1, M2]
    ) = Option.some(f))
    functor_new_dst_cat(product_category(c, d), c, product_fst_obj[O1, O2], product_fst_mor[M1, M2], f)
}

/// The first projection functor has `product_fst_obj[O1, O2]` as its object map.
theorem product_fst_functor_obj_map[O1, M1, O2, M2](
    c: Category[O1, M1], d: Category[O2, M2]
) {
    product_fst_functor(c, d).obj_map = product_fst_obj[O1, O2]
} by {
    let f = product_fst_functor(c, d)
    (Functor[Pair[O1, O2], Pair[M1, M2], O1, M1].new(
        product_category(c, d), c, product_fst_obj[O1, O2], product_fst_mor[M1, M2]
    ) = Option.some(f))
    functor_new_obj_map(product_category(c, d), c, product_fst_obj[O1, O2], product_fst_mor[M1, M2], f)
}

/// The first projection functor has `product_fst_mor[M1, M2]` as its morphism map.
theorem product_fst_functor_mor_map[O1, M1, O2, M2](
    c: Category[O1, M1], d: Category[O2, M2]
) {
    product_fst_functor(c, d).mor_map = product_fst_mor[M1, M2]
} by {
    let f = product_fst_functor(c, d)
    (Functor[Pair[O1, O2], Pair[M1, M2], O1, M1].new(
        product_category(c, d), c, product_fst_obj[O1, O2], product_fst_mor[M1, M2]
    ) = Option.some(f))
    functor_new_mor_map(product_category(c, d), c, product_fst_obj[O1, O2], product_fst_mor[M1, M2], f)
}

/// The second projection functor has the product category as source.
theorem product_snd_functor_src_cat[O1, M1, O2, M2](
    c: Category[O1, M1], d: Category[O2, M2]
) {
    product_snd_functor(c, d).src_cat = product_category(c, d)
} by {
    let f = product_snd_functor(c, d)
    (Functor[Pair[O1, O2], Pair[M1, M2], O2, M2].new(
        product_category(c, d), d, product_snd_obj[O1, O2], product_snd_mor[M1, M2]
    ) = Option.some(f))
    functor_new_src_cat(product_category(c, d), d, product_snd_obj[O1, O2], product_snd_mor[M1, M2], f)
}

/// The second projection functor has the second category as target.
theorem product_snd_functor_dst_cat[O1, M1, O2, M2](
    c: Category[O1, M1], d: Category[O2, M2]
) {
    product_snd_functor(c, d).dst_cat = d
} by {
    let f = product_snd_functor(c, d)
    (Functor[Pair[O1, O2], Pair[M1, M2], O2, M2].new(
        product_category(c, d), d, product_snd_obj[O1, O2], product_snd_mor[M1, M2]
    ) = Option.some(f))
    functor_new_dst_cat(product_category(c, d), d, product_snd_obj[O1, O2], product_snd_mor[M1, M2], f)
}

/// The second projection functor has `product_snd_obj[O1, O2]` as its object map.
theorem product_snd_functor_obj_map[O1, M1, O2, M2](
    c: Category[O1, M1], d: Category[O2, M2]
) {
    product_snd_functor(c, d).obj_map = product_snd_obj[O1, O2]
} by {
    let f = product_snd_functor(c, d)
    (Functor[Pair[O1, O2], Pair[M1, M2], O2, M2].new(
        product_category(c, d), d, product_snd_obj[O1, O2], product_snd_mor[M1, M2]
    ) = Option.some(f))
    functor_new_obj_map(product_category(c, d), d, product_snd_obj[O1, O2], product_snd_mor[M1, M2], f)
}

/// The second projection functor has `product_snd_mor[M1, M2]` as its morphism map.
theorem product_snd_functor_mor_map[O1, M1, O2, M2](
    c: Category[O1, M1], d: Category[O2, M2]
) {
    product_snd_functor(c, d).mor_map = product_snd_mor[M1, M2]
} by {
    let f = product_snd_functor(c, d)
    (Functor[Pair[O1, O2], Pair[M1, M2], O2, M2].new(
        product_category(c, d), d, product_snd_obj[O1, O2], product_snd_mor[M1, M2]
    ) = Option.some(f))
    functor_new_mor_map(product_category(c, d), d, product_snd_obj[O1, O2], product_snd_mor[M1, M2], f)
}

/// The first projection functor evaluates on objects by taking first components.
theorem product_fst_functor_obj_map_apply[O1, M1, O2, M2](
    c: Category[O1, M1], d: Category[O2, M2], x: Pair[O1, O2]
) {
    product_fst_functor(c, d).obj_map(x) = x.first
} by {
    product_fst_functor_obj_map(c, d)
    product_fst_functor(c, d).obj_map = product_fst_obj[O1, O2]
    product_fst_obj[O1, O2](x) = x.first
    product_fst_functor(c, d).obj_map(x) = x.first
}

/// The first projection functor evaluates on morphisms by taking first components.
theorem product_fst_functor_mor_map_apply[O1, M1, O2, M2](
    c: Category[O1, M1], d: Category[O2, M2], f: Pair[M1, M2]
) {
    product_fst_functor(c, d).mor_map(f) = f.first
} by {
    product_fst_functor_mor_map(c, d)
    product_fst_functor(c, d).mor_map = product_fst_mor[M1, M2]
    product_fst_mor[M1, M2](f) = f.first
    product_fst_functor(c, d).mor_map(f) = f.first
}

/// The second projection functor evaluates on objects by taking second components.
theorem product_snd_functor_obj_map_apply[O1, M1, O2, M2](
    c: Category[O1, M1], d: Category[O2, M2], x: Pair[O1, O2]
) {
    product_snd_functor(c, d).obj_map(x) = x.second
} by {
    product_snd_functor_obj_map(c, d)
    product_snd_functor(c, d).obj_map = product_snd_obj[O1, O2]
    product_snd_obj[O1, O2](x) = x.second
    product_snd_functor(c, d).obj_map(x) = x.second
}

/// The second projection functor evaluates on morphisms by taking second components.
theorem product_snd_functor_mor_map_apply[O1, M1, O2, M2](
    c: Category[O1, M1], d: Category[O2, M2], f: Pair[M1, M2]
) {
    product_snd_functor(c, d).mor_map(f) = f.second
} by {
    product_snd_functor_mor_map(c, d)
    product_snd_functor(c, d).mor_map = product_snd_mor[M1, M2]
    product_snd_mor[M1, M2](f) = f.second
    product_snd_functor(c, d).mor_map(f) = f.second
}

/// The two projection functors have the same product-category source.
theorem product_projection_functors_common_src[O1, M1, O2, M2](
    c: Category[O1, M1], d: Category[O2, M2]
) {
    product_fst_functor(c, d).src_cat = product_snd_functor(c, d).src_cat
} by {
    product_fst_functor_src_cat(c, d)
    product_snd_functor_src_cat(c, d)
    product_fst_functor(c, d).src_cat = product_category(c, d)
    product_snd_functor(c, d).src_cat = product_category(c, d)
    product_fst_functor(c, d).src_cat = product_snd_functor(c, d).src_cat
}
