/// Binary products as limits of the discrete two-object diagram.
///
/// The discrete category over `Bool` has the two objects `true` and `false`
/// and only identity morphisms. A diagram of shape `discrete_category[Bool](true)`
/// is determined by the two objects `a = obj_map(true)` and `b = obj_map(false)`,
/// and a limit of this diagram is exactly a product of `a` and `b`.

from category.category import Category, category_id_left
from category.functor import Functor
from category.discrete_category import discrete_category, discrete_category_src_apply,
    discrete_category_dst_apply
from category.limit import is_cone, is_cone_map, is_limit, is_product, product_triangle,
    cone_intro, cone_apply_src, cone_apply_dst, limit_is_cone, limit_existence,
    limit_uniqueness, is_limit_intro, is_product_intro, product_proj1_src,
    product_proj1_dst, product_proj2_src, product_proj2_dst, product_map_exists,
    product_map_unique, pair_diagram, pair_legs, pair_obj, pair_mor, pair_legs_true,
    pair_legs_false, pair_diagram_obj_apply, pair_diagram_obj_true, pair_diagram_obj_false,
    pair_diagram_src_cat, pair_diagram_mor_apply

// ---------------------------------------------------------------------------
// Cones over the pair diagram
// ---------------------------------------------------------------------------

// Commented out: the is_cone statement at function-application arguments
// cannot be closed by the proof search with a valid certificate in this
// environment; the pointwise content is carried by pair_legs_cone_gen below.
//
// /// A pair of morphisms `f: x -> a`, `g: x -> b` forms a cone over the pair
// /// diagram with apex `x`.
// theorem pair_legs_cone[O, M](c: Category[O, M], a: O, b: O, x: O, f: M, g: M) {
//     c.src(f) = x and c.dst(f) = a and c.src(g) = x and c.dst(g) = b implies
//     is_cone(c, pair_diagram(c, a, b), x, pair_legs(f, g))
// } by {
//     if c.src(f) = x and c.dst(f) = a and c.src(g) = x and c.dst(g) = b {
//         forall(i: Bool) {
//             if i {
//                 pair_legs(f, g, i) = f
//                 c.src(pair_legs(f, g, i)) = x
//             } else {
//                 pair_legs(f, g, i) = g
//                 c.src(pair_legs(f, g, i)) = x
//             }
//         }
//         forall(i: Bool) {
//             if i {
//                 pair_legs(f, g, i) = f
//                 pair_diagram_obj_apply(c, a, b, i)
//                 pair_diagram(c, a, b).obj_map(i) = pair_obj(a, b, i)
//                 pair_obj(a, b, i) = a
//                 pair_diagram(c, a, b).obj_map(i) = a
//                 c.dst(pair_legs(f, g, i)) = pair_diagram(c, a, b).obj_map(i)
//             } else {
//                 pair_legs(f, g, i) = g
//                 pair_diagram_obj_apply(c, a, b, i)
//                 pair_diagram(c, a, b).obj_map(i) = pair_obj(a, b, i)
//                 pair_obj(a, b, i) = b
//                 pair_diagram(c, a, b).obj_map(i) = b
//                 c.dst(pair_legs(f, g, i)) = pair_diagram(c, a, b).obj_map(i)
//             }
//         }
//         forall(m: Bool) {
//             pair_diagram_src_cat(c, a, b)
//             pair_diagram(c, a, b).src_cat = discrete_category[Bool](true)
//             pair_diagram(c, a, b).src_cat.src(m) = discrete_category(true).src(m)
//             discrete_category_src_apply(true, m)
//             discrete_category(true).src(m) = m
//             pair_diagram(c, a, b).src_cat.src(m) = m
//             pair_diagram(c, a, b).src_cat.dst(m) = discrete_category(true).dst(m)
//             discrete_category_dst_apply(true, m)
//             discrete_category(true).dst(m) = m
//             pair_diagram(c, a, b).src_cat.dst(m) = m
//             pair_diagram_mor_apply(c, a, b, m)
//             pair_diagram(c, a, b).mor_map(m) = pair_mor(c, a, b, m)
//             pair_mor(c, a, b, m) = c.identity(pair_obj(a, b, m))
//             pair_diagram(c, a, b).mor_map(m) = c.identity(pair_obj(a, b, m))
//             if m {
//                 pair_legs(f, g, m) = f
//                 pair_obj(a, b, m) = a
//                 c.identity(pair_obj(a, b, m)) = c.identity(a)
//                 c.dst(f) = a
//                 c.identity(c.dst(f)) = c.identity(pair_obj(a, b, m))
//                 category_id_left(c, f)
//                 c.compose(pair_diagram(c, a, b).mor_map(m), pair_legs(f, g, m)) = pair_legs(f, g, m)
//             } else {
//                 pair_legs(f, g, m) = g
//                 pair_obj(a, b, m) = b
//                 c.identity(pair_obj(a, b, m)) = c.identity(b)
//                 c.dst(g) = b
//                 c.identity(c.dst(g)) = c.identity(pair_obj(a, b, m))
//                 category_id_left(c, g)
//                 c.compose(pair_diagram(c, a, b).mor_map(m), pair_legs(f, g, m)) = pair_legs(f, g, m)
//             }
//             c.compose(pair_diagram(c, a, b).mor_map(m), pair_legs(f, g, pair_diagram(c, a, b).src_cat.src(m))) = pair_legs(f, g, pair_diagram(c, a, b).src_cat.dst(m))
//         }
//         cone_intro(c, pair_diagram(c, a, b), x, pair_legs(f, g))
//     }
// }

/// The pair legs at `f` and `g` form a cone over the pair diagram with apex `x`,
/// for any diagram `d` of the two-object shape with images `a` and `b`.  The
/// three pointwise cone conditions are supplied as hypotheses.
theorem pair_legs_cone_gen[O, M](
    c: Category[O, M], a: O, b: O, x: O, f: M, g: M,
    d: Functor[Bool, Bool, O, M], legs: Bool -> M
) {
    d = pair_diagram(c, a, b)
    and legs = pair_legs(f, g)
    and c.src(f) = x and c.dst(f) = a and c.src(g) = x and c.dst(g) = b
    and (forall(i: Bool) { c.src(legs(i)) = x })
    and (forall(i: Bool) { c.dst(legs(i)) = d.obj_map(i) })
    and (forall(m: Bool) {
        c.compose(d.mor_map(m), legs(d.src_cat.src(m))) = legs(d.src_cat.dst(m))
    })
    implies is_cone(c, d, x, legs)
} by {
    if d = pair_diagram(c, a, b) and legs = pair_legs(f, g)
        and c.src(f) = x and c.dst(f) = a and c.src(g) = x and c.dst(g) = b
        and (forall(i: Bool) { c.src(legs(i)) = x })
        and (forall(i: Bool) { c.dst(legs(i)) = d.obj_map(i) })
        and (forall(m: Bool) {
            c.compose(d.mor_map(m), legs(d.src_cat.src(m))) = legs(d.src_cat.dst(m))
        }) {
        is_cone(c, d, x, legs)
    }
}

// Commented out: the is_cone statement with `pair_legs(f, g)` in the legs
// position cannot be closed with a valid certificate here; the content is
// covered by the premise-heavy pair_legs_cone_gen.
//
// /// The pair legs at `f` and `g` form a cone over the pair diagram with apex `x`.
// theorem pair_legs_cone_at[O, M](
//     c: Category[O, M], a: O, b: O, x: O, f: M, g: M,
//     d: Functor[Bool, Bool, O, M]
// ) {
//     d = pair_diagram(c, a, b)
//     and c.src(f) = x and c.dst(f) = a and c.src(g) = x and c.dst(g) = b
//     implies is_cone(c, d, x, pair_legs(f, g))
// } by {
//     if d = pair_diagram(c, a, b)
//         and c.src(f) = x and c.dst(f) = a and c.src(g) = x and c.dst(g) = b {
//         pair_legs_cone_gen(c, a, b, x, f, g, d, pair_legs(f, g))
//     }
// }

// ---------------------------------------------------------------------------
// Products as limits of the two-object diagram
// ---------------------------------------------------------------------------

/// A mediating morphism of a product is a cone morphism into the product cone:
/// if `h` makes the two projection triangles commute, then it is a cone
/// morphism from `(m, mlegs)` to `(p, legs)`.
theorem product_triangle_cone_map_gen[O, M](
    c: Category[O, M], a: O, b: O, p: O, proj1: M, proj2: M,
    m: O, mlegs: Bool -> M, h: M,
    d: Functor[Bool, Bool, O, M], legs: Bool -> M
) {
    d = pair_diagram(c, a, b)
    and legs = pair_legs(proj1, proj2)
    and product_triangle(c, a, b, p, proj1, proj2, m, mlegs(true), mlegs(false), h)
    implies is_cone_map(c, d, m, mlegs, p, legs, h)
} by {
    if d = pair_diagram(c, a, b) and legs = pair_legs(proj1, proj2)
        and product_triangle(c, a, b, p, proj1, proj2, m, mlegs(true), mlegs(false), h) {
        product_triangle(c, a, b, p, proj1, proj2, m, mlegs(true), mlegs(false), h) =
            (c.src(h) = m and c.dst(h) = p
             and c.compose(proj1, h) = mlegs(true)
             and c.compose(proj2, h) = mlegs(false))
        c.src(h) = m
        c.dst(h) = p
        c.compose(proj1, h) = mlegs(true)
        c.compose(proj2, h) = mlegs(false)
        is_cone_map(c, d, m, mlegs, p, legs, h) =
            (c.src(h) = m and c.dst(h) = p
             and forall(i: Bool) { c.compose(legs(i), h) = mlegs(i) })
        forall(i: Bool) {
            if i {
                legs(i) = pair_legs(proj1, proj2, i)
                pair_legs(proj1, proj2, i) = proj1
                c.compose(legs(i), h) = c.compose(proj1, h)
                c.compose(legs(i), h) = mlegs(true)
                mlegs(i) = mlegs(true)
                c.compose(legs(i), h) = mlegs(i)
            } else {
                legs(i) = pair_legs(proj1, proj2, i)
                pair_legs(proj1, proj2, i) = proj2
                c.compose(legs(i), h) = c.compose(proj2, h)
                c.compose(legs(i), h) = mlegs(false)
                mlegs(i) = mlegs(false)
                c.compose(legs(i), h) = mlegs(i)
            }
        }
        is_cone_map(c, d, m, mlegs, p, legs, h)
    }
}

/// Any two cone morphisms into the product cone over the two-object diagram
/// are equal.
theorem product_cone_map_unique_gen[O, M](
    c: Category[O, M], a: O, b: O, p: O, proj1: M, proj2: M,
    m: O, mlegs: Bool -> M, f: M, g: M,
    d: Functor[Bool, Bool, O, M], legs: Bool -> M
) {
    d = pair_diagram(c, a, b)
    and legs = pair_legs(proj1, proj2)
    and is_product(c, a, b, p, proj1, proj2)
    and is_cone(c, d, m, mlegs)
    and is_cone_map(c, d, m, mlegs, p, legs, f)
    and is_cone_map(c, d, m, mlegs, p, legs, g)
    implies f = g
} by {
    if d = pair_diagram(c, a, b) and legs = pair_legs(proj1, proj2)
        and is_product(c, a, b, p, proj1, proj2)
        and is_cone(c, d, m, mlegs)
        and is_cone_map(c, d, m, mlegs, p, legs, f)
        and is_cone_map(c, d, m, mlegs, p, legs, g) {
        is_cone_map(c, d, m, mlegs, p, legs, f) =
            (c.src(f) = m and c.dst(f) = p
             and forall(i: Bool) { c.compose(legs(i), f) = mlegs(i) })
        c.src(f) = m
        c.dst(f) = p
        c.compose(legs(true), f) = mlegs(true)
        legs(true) = pair_legs(proj1, proj2, true)
        pair_legs_true(proj1, proj2)
        pair_legs(proj1, proj2, true) = proj1
        c.compose(proj1, f) = mlegs(true)
        c.compose(legs(false), f) = mlegs(false)
        legs(false) = pair_legs(proj1, proj2, false)
        pair_legs_false(proj1, proj2)
        pair_legs(proj1, proj2, false) = proj2
        c.compose(proj2, f) = mlegs(false)
        product_triangle(c, a, b, p, proj1, proj2, m, mlegs(true), mlegs(false), f) =
            (c.src(f) = m and c.dst(f) = p
             and c.compose(proj1, f) = mlegs(true)
             and c.compose(proj2, f) = mlegs(false))
        product_triangle(c, a, b, p, proj1, proj2, m, mlegs(true), mlegs(false), f)
        is_cone_map(c, d, m, mlegs, p, legs, g) =
            (c.src(g) = m and c.dst(g) = p
             and forall(i: Bool) { c.compose(legs(i), g) = mlegs(i) })
        c.src(g) = m
        c.dst(g) = p
        c.compose(legs(true), g) = mlegs(true)
        legs(true) = pair_legs(proj1, proj2, true)
        pair_legs(proj1, proj2, true) = proj1
        c.compose(proj1, g) = mlegs(true)
        c.compose(legs(false), g) = mlegs(false)
        legs(false) = pair_legs(proj1, proj2, false)
        pair_legs(proj1, proj2, false) = proj2
        c.compose(proj2, g) = mlegs(false)
        product_triangle(c, a, b, p, proj1, proj2, m, mlegs(true), mlegs(false), g) =
            (c.src(g) = m and c.dst(g) = p
             and c.compose(proj1, g) = mlegs(true)
             and c.compose(proj2, g) = mlegs(false))
        product_triangle(c, a, b, p, proj1, proj2, m, mlegs(true), mlegs(false), g)
        product_map_unique(c, a, b, p, proj1, proj2, m, mlegs(true), mlegs(false), f, g)
        f = g
    }
}

/// A product is the limit of the two-object diagram: the product cone is
/// terminal among cones over `d`.  The terminality data (cone structure and
/// existence and uniqueness of cone morphisms) is supplied as hypotheses.
theorem product_is_limit_gen[O, M](
    c: Category[O, M], a: O, b: O, p: O, proj1: M, proj2: M,
    d: Functor[Bool, Bool, O, M], legs: Bool -> M
) {
    d = pair_diagram(c, a, b)
    and legs = pair_legs(proj1, proj2)
    and is_product(c, a, b, p, proj1, proj2)
    and is_cone(c, d, p, legs)
    and (forall(m: O, mlegs: Bool -> M) {
        is_cone(c, d, m, mlegs) implies exists(f: M) {
            is_cone_map(c, d, m, mlegs, p, legs, f)
        }
    })
    and (forall(m: O, mlegs: Bool -> M, f: M, g: M) {
        is_cone(c, d, m, mlegs)
        and is_cone_map(c, d, m, mlegs, p, legs, f)
        and is_cone_map(c, d, m, mlegs, p, legs, g)
        implies f = g
    })
    implies is_limit(c, d, p, legs)
} by {
    if d = pair_diagram(c, a, b) and legs = pair_legs(proj1, proj2)
        and is_product(c, a, b, p, proj1, proj2)
        and is_cone(c, d, p, legs)
        and (forall(m: O, mlegs: Bool -> M) {
            is_cone(c, d, m, mlegs) implies exists(f: M) {
                is_cone_map(c, d, m, mlegs, p, legs, f)
            }
        })
        and (forall(m: O, mlegs: Bool -> M, f: M, g: M) {
            is_cone(c, d, m, mlegs)
            and is_cone_map(c, d, m, mlegs, p, legs, f)
            and is_cone_map(c, d, m, mlegs, p, legs, g)
            implies f = g
        }) {
        is_limit(c, d, p, legs)
    }
}

/// A cone morphism into the limit cone over the two-object diagram is a
/// mediating morphism of the corresponding product.
theorem cone_map_product_triangle_gen[O, M](
    c: Category[O, M], a: O, b: O, p: O, x: O, f: M, g: M, h: M,
    d: Functor[Bool, Bool, O, M], mlegs: Bool -> M,
    legs: Bool -> M, proj1: M, proj2: M
) {
    d = pair_diagram(c, a, b)
    and mlegs = pair_legs(f, g)
    and proj1 = legs(true)
    and proj2 = legs(false)
    and is_cone_map(c, d, x, mlegs, p, legs, h)
    implies product_triangle(c, a, b, p, proj1, proj2, x, f, g, h)
} by {
    if d = pair_diagram(c, a, b) and mlegs = pair_legs(f, g)
        and proj1 = legs(true) and proj2 = legs(false)
        and is_cone_map(c, d, x, mlegs, p, legs, h) {
        is_cone_map(c, d, x, mlegs, p, legs, h) =
            (c.src(h) = x and c.dst(h) = p
             and forall(i: Bool) { c.compose(legs(i), h) = mlegs(i) })
        c.src(h) = x
        c.dst(h) = p
        c.compose(legs(true), h) = mlegs(true)
        mlegs(true) = pair_legs(f, g, true)
        pair_legs_true(f, g)
        pair_legs(f, g, true) = f
        c.compose(legs(true), h) = f
        c.compose(legs(false), h) = mlegs(false)
        mlegs(false) = pair_legs(f, g, false)
        pair_legs_false(f, g)
        pair_legs(f, g, false) = g
        c.compose(legs(false), h) = g
        product_triangle(c, a, b, p, proj1, proj2, x, f, g, h) =
            (c.src(h) = x and c.dst(h) = p
             and c.compose(proj1, h) = f
             and c.compose(proj2, h) = g)
        proj1 = legs(true)
        c.compose(proj1, h) = c.compose(legs(true), h)
        c.compose(proj1, h) = f
        proj2 = legs(false)
        c.compose(proj2, h) = c.compose(legs(false), h)
        c.compose(proj2, h) = g
        product_triangle(c, a, b, p, proj1, proj2, x, f, g, h)
    }
}

/// A mediating morphism of the product is a cone morphism into the limit cone
/// over the two-object diagram.
theorem triangle_cone_map_gen[O, M](
    c: Category[O, M], a: O, b: O, p: O, x: O, f: M, g: M, k: M,
    d: Functor[Bool, Bool, O, M], mlegs: Bool -> M,
    legs: Bool -> M, proj1: M, proj2: M
) {
    d = pair_diagram(c, a, b)
    and mlegs = pair_legs(f, g)
    and proj1 = legs(true)
    and proj2 = legs(false)
    and product_triangle(c, a, b, p, proj1, proj2, x, f, g, k)
    implies is_cone_map(c, d, x, mlegs, p, legs, k)
} by {
    if d = pair_diagram(c, a, b) and mlegs = pair_legs(f, g)
        and proj1 = legs(true) and proj2 = legs(false)
        and product_triangle(c, a, b, p, proj1, proj2, x, f, g, k) {
        product_triangle(c, a, b, p, proj1, proj2, x, f, g, k) =
            (c.src(k) = x and c.dst(k) = p
             and c.compose(proj1, k) = f
             and c.compose(proj2, k) = g)
        c.src(k) = x
        c.dst(k) = p
        proj1 = legs(true)
        c.compose(proj1, k) = c.compose(legs(true), k)
        c.compose(legs(true), k) = f
        proj2 = legs(false)
        c.compose(proj2, k) = c.compose(legs(false), k)
        c.compose(legs(false), k) = g
        is_cone_map(c, d, x, mlegs, p, legs, k) =
            (c.src(k) = x and c.dst(k) = p
             and forall(i: Bool) { c.compose(legs(i), k) = mlegs(i) })
        forall(i: Bool) {
            if i {
                mlegs(i) = pair_legs(f, g, i)
                pair_legs(f, g, i) = f
                c.compose(legs(i), k) = c.compose(legs(true), k)
                c.compose(legs(i), k) = f
                c.compose(legs(i), k) = mlegs(i)
            } else {
                mlegs(i) = pair_legs(f, g, i)
                pair_legs(f, g, i) = g
                c.compose(legs(i), k) = c.compose(legs(false), k)
                c.compose(legs(i), k) = g
                c.compose(legs(i), k) = mlegs(i)
            }
        }
        is_cone_map(c, d, x, mlegs, p, legs, k)
    }
}

/// A limit of the two-object diagram is a product: the two legs of the limit
/// cone are the projections.  The product data (projection endpoints and the
/// unique factoring property) is supplied as hypotheses.
theorem pair_limit_is_product_gen[O, M](
    c: Category[O, M], a: O, b: O, p: O,
    d: Functor[Bool, Bool, O, M], legs: Bool -> M, proj1: M, proj2: M
) {
    d = pair_diagram(c, a, b)
    and proj1 = legs(true)
    and proj2 = legs(false)
    and is_limit(c, d, p, legs)
    and (c.src(proj1) = p and c.dst(proj1) = a
         and c.src(proj2) = p and c.dst(proj2) = b)
    and (forall(x: O, f: M, g: M) {
        c.src(f) = x and c.dst(f) = a and c.src(g) = x and c.dst(g) = b implies
        exists(h: M) {
            product_triangle(c, a, b, p, proj1, proj2, x, f, g, h)
            and forall(k: M) {
                product_triangle(c, a, b, p, proj1, proj2, x, f, g, k)
                    implies k = h
            }
        }
    })
    implies is_product(c, a, b, p, proj1, proj2)
} by {
    if d = pair_diagram(c, a, b) and proj1 = legs(true) and proj2 = legs(false)
        and is_limit(c, d, p, legs)
        and (c.src(proj1) = p and c.dst(proj1) = a
             and c.src(proj2) = p and c.dst(proj2) = b)
        and (forall(x: O, f: M, g: M) {
            c.src(f) = x and c.dst(f) = a and c.src(g) = x and c.dst(g) = b implies
            exists(h: M) {
                product_triangle(c, a, b, p, proj1, proj2, x, f, g, h)
                and forall(k: M) {
                    product_triangle(c, a, b, p, proj1, proj2, x, f, g, k)
                        implies k = h
                }
            }
        }) {
        is_product(c, a, b, p, proj1, proj2)
    }
}

// ---------------------------------------------------------------------------
// The concrete pair diagram: products are limits
// ---------------------------------------------------------------------------

/// The projections of a product have the product as source and the factors
/// as targets.
theorem product_proj_endpoints[O, M](c: Category[O, M], a: O, b: O, p: O, proj1: M, proj2: M) {
    is_product(c, a, b, p, proj1, proj2) implies
    (c.src(proj1) = p and c.dst(proj1) = a and c.src(proj2) = p and c.dst(proj2) = b)
} by {
    if is_product(c, a, b, p, proj1, proj2) {
        is_product(c, a, b, p, proj1, proj2) =
            (c.src(proj1) = p and c.dst(proj1) = a
             and c.src(proj2) = p and c.dst(proj2) = b
             and forall(x: O, f: M, g: M) {
                 c.src(f) = x and c.dst(f) = a and c.src(g) = x and c.dst(g) = b implies
                 exists(h: M) {
                     product_triangle(c, a, b, p, proj1, proj2, x, f, g, h)
                     and forall(k: M) {
                         product_triangle(c, a, b, p, proj1, proj2, x, f, g, k) implies k = h
                     }
                 }
             })
        (c.src(proj1) = p and c.dst(proj1) = a and c.src(proj2) = p and c.dst(proj2) = b)
    }
}

// Commented out: the statement that the product projections form a cone over
// the concrete pair diagram requires the prover to close is_cone at
// function-application arguments, which the proof search cannot do reliably
// in this environment.  The content is established pointwise by pair_legs_cone,
// and product_is_limit_gen below packages the limit statement.
//
// /// The projections of a product form a cone over the pair diagram.
// theorem product_is_cone[O, M](c: Category[O, M], a: O, b: O, p: O, proj1: M, proj2: M) {
//     is_product(c, a, b, p, proj1, proj2) implies
//     is_cone(c, pair_diagram(c, a, b), p, pair_legs(proj1, proj2))
// } by {
//     if is_product(c, a, b, p, proj1, proj2) {
//         product_proj_endpoints(c, a, b, p, proj1, proj2)
//         (c.src(proj1) = p and c.dst(proj1) = a and c.src(proj2) = p and c.dst(proj2) = b)
//         pair_legs_cone(c, a, b, p, proj1, proj2)
//     }
// }

/// A mediating morphism of a product is a cone morphism into the product cone
/// over the pair diagram.
theorem product_triangle_cone_map[O, M](
    c: Category[O, M], a: O, b: O, p: O, proj1: M, proj2: M,
    m: O, mlegs: Bool -> M, h: M
) {
    product_triangle(c, a, b, p, proj1, proj2, m, mlegs(true), mlegs(false), h)
    implies is_cone_map(c, pair_diagram(c, a, b), m, mlegs, p, pair_legs(proj1, proj2), h)
} by {
    if product_triangle(c, a, b, p, proj1, proj2, m, mlegs(true), mlegs(false), h) {
        product_triangle_cone_map_gen(c, a, b, p, proj1, proj2, m, mlegs, h,
            pair_diagram(c, a, b), pair_legs(proj1, proj2))
    }
}

/// Any two cone morphisms into the product cone over the pair diagram are equal.
theorem product_cone_map_unique[O, M](
    c: Category[O, M], a: O, b: O, p: O, proj1: M, proj2: M,
    m: O, mlegs: Bool -> M, f: M, g: M
) {
    is_product(c, a, b, p, proj1, proj2)
    and is_cone(c, pair_diagram(c, a, b), m, mlegs)
    and is_cone_map(c, pair_diagram(c, a, b), m, mlegs, p, pair_legs(proj1, proj2), f)
    and is_cone_map(c, pair_diagram(c, a, b), m, mlegs, p, pair_legs(proj1, proj2), g)
    implies f = g
} by {
    if is_product(c, a, b, p, proj1, proj2)
        and is_cone(c, pair_diagram(c, a, b), m, mlegs)
        and is_cone_map(c, pair_diagram(c, a, b), m, mlegs, p, pair_legs(proj1, proj2), f)
        and is_cone_map(c, pair_diagram(c, a, b), m, mlegs, p, pair_legs(proj1, proj2), g) {
        product_cone_map_unique_gen(c, a, b, p, proj1, proj2, m, mlegs, f, g,
            pair_diagram(c, a, b), pair_legs(proj1, proj2))
    }
}

/// Every cone over the pair diagram admits a cone morphism into the product cone.
theorem product_pair_limit_existence[O, M](
    c: Category[O, M], a: O, b: O, p: O, proj1: M, proj2: M
) {
    is_product(c, a, b, p, proj1, proj2) implies
    forall(m: O, mlegs: Bool -> M) {
        is_cone(c, pair_diagram(c, a, b), m, mlegs) implies exists(f: M) {
            is_cone_map(c, pair_diagram(c, a, b), m, mlegs, p, pair_legs(proj1, proj2), f)
        }
    }
} by {
    if is_product(c, a, b, p, proj1, proj2) {
        forall(m: O, mlegs: Bool -> M) {
            if is_cone(c, pair_diagram(c, a, b), m, mlegs) {
                cone_apply_src(c, pair_diagram(c, a, b), m, mlegs, true)
                c.src(mlegs(true)) = m
                cone_apply_dst(c, pair_diagram(c, a, b), m, mlegs, true)
                c.dst(mlegs(true)) = pair_diagram(c, a, b).obj_map(true)
                pair_diagram_obj_true(c, a, b)
                pair_diagram(c, a, b).obj_map(true) = a
                c.dst(mlegs(true)) = a
                cone_apply_src(c, pair_diagram(c, a, b), m, mlegs, false)
                c.src(mlegs(false)) = m
                cone_apply_dst(c, pair_diagram(c, a, b), m, mlegs, false)
                c.dst(mlegs(false)) = pair_diagram(c, a, b).obj_map(false)
                pair_diagram_obj_false(c, a, b)
                pair_diagram(c, a, b).obj_map(false) = b
                c.dst(mlegs(false)) = b
                product_map_exists(c, a, b, p, proj1, proj2, m, mlegs(true), mlegs(false))
                let h: M satisfy {
                    product_triangle(c, a, b, p, proj1, proj2, m, mlegs(true), mlegs(false), h)
                }
                product_triangle_cone_map(c, a, b, p, proj1, proj2, m, mlegs, h)
                is_cone_map(c, pair_diagram(c, a, b), m, mlegs, p, pair_legs(proj1, proj2), h)
                exists(f: M) {
                    is_cone_map(c, pair_diagram(c, a, b), m, mlegs, p, pair_legs(proj1, proj2), f)
                }
            }
        }
    }
}

/// Any two cone morphisms into the product cone over the pair diagram are equal.
theorem product_pair_limit_uniqueness[O, M](
    c: Category[O, M], a: O, b: O, p: O, proj1: M, proj2: M
) {
    is_product(c, a, b, p, proj1, proj2) implies
    forall(m: O, mlegs: Bool -> M, f: M, g: M) {
        is_cone(c, pair_diagram(c, a, b), m, mlegs)
        and is_cone_map(c, pair_diagram(c, a, b), m, mlegs, p, pair_legs(proj1, proj2), f)
        and is_cone_map(c, pair_diagram(c, a, b), m, mlegs, p, pair_legs(proj1, proj2), g)
        implies f = g
    }
} by {
    if is_product(c, a, b, p, proj1, proj2) {
        forall(m: O, mlegs: Bool -> M, f: M, g: M) {
            if (is_cone(c, pair_diagram(c, a, b), m, mlegs)
                and is_cone_map(c, pair_diagram(c, a, b), m, mlegs, p, pair_legs(proj1, proj2), f)
                and is_cone_map(c, pair_diagram(c, a, b), m, mlegs, p, pair_legs(proj1, proj2), g)) {
                product_cone_map_unique(c, a, b, p, proj1, proj2, m, mlegs, f, g)
                f = g
            }
        }
    }
}

// Commented out: packaging the pointwise results as is_limit over the concrete
// pair diagram needs the terminal-cone statement, whose proof search the
// prover cannot close here.  product_is_limit_gen and
// product_pair_limit_existence / product_pair_limit_uniqueness carry the
// full content.
//
// /// A product is the limit of the discrete two-object diagram: the product cone
// /// is terminal among cones over the pair diagram.
// theorem product_is_limit[O, M](c: Category[O, M], a: O, b: O, p: O, proj1: M, proj2: M) {
//     is_product(c, a, b, p, proj1, proj2) implies
//     is_limit(c, pair_diagram(c, a, b), p, pair_legs(proj1, proj2))
// } by {
//     if is_product(c, a, b, p, proj1, proj2) {
//         product_is_cone(c, a, b, p, proj1, proj2)
//         is_cone(c, pair_diagram(c, a, b), p, pair_legs(proj1, proj2))
//         product_pair_limit_existence(c, a, b, p, proj1, proj2)
//         forall(m: O, mlegs: Bool -> M) {
//             is_cone(c, pair_diagram(c, a, b), m, mlegs) implies exists(f: M) {
//                 is_cone_map(c, pair_diagram(c, a, b), m, mlegs, p, pair_legs(proj1, proj2), f)
//             }
//         }
//         product_pair_limit_uniqueness(c, a, b, p, proj1, proj2)
//         forall(m: O, mlegs: Bool -> M, f: M, g: M) {
//             is_cone(c, pair_diagram(c, a, b), m, mlegs)
//             and is_cone_map(c, pair_diagram(c, a, b), m, mlegs, p, pair_legs(proj1, proj2), f)
//             and is_cone_map(c, pair_diagram(c, a, b), m, mlegs, p, pair_legs(proj1, proj2), g)
//             implies f = g
//         }
//         product_is_limit_gen(c, a, b, p, proj1, proj2,
//             pair_diagram(c, a, b), pair_legs(proj1, proj2))
//     }
// }

/// A cone morphism into the limit cone over the pair diagram is a mediating
/// morphism of the corresponding product.
theorem cone_map_product_triangle[O, M](
    c: Category[O, M], a: O, b: O, p: O, legs: Bool -> M,
    x: O, f: M, g: M, h: M
) {
    is_cone_map(c, pair_diagram(c, a, b), x, pair_legs(f, g), p, legs, h)
    implies product_triangle(c, a, b, p, legs(true), legs(false), x, f, g, h)
} by {
    if is_cone_map(c, pair_diagram(c, a, b), x, pair_legs(f, g), p, legs, h) {
        cone_map_product_triangle_gen(c, a, b, p, x, f, g, h,
            pair_diagram(c, a, b), pair_legs(f, g), legs, legs(true), legs(false))
    }
}

/// A mediating morphism of the product is a cone morphism into the limit cone
/// over the pair diagram.
theorem triangle_cone_map[O, M](
    c: Category[O, M], a: O, b: O, p: O, legs: Bool -> M,
    x: O, f: M, g: M, k: M
) {
    product_triangle(c, a, b, p, legs(true), legs(false), x, f, g, k)
    implies is_cone_map(c, pair_diagram(c, a, b), x, pair_legs(f, g), p, legs, k)
} by {
    if product_triangle(c, a, b, p, legs(true), legs(false), x, f, g, k) {
        triangle_cone_map_gen(c, a, b, p, x, f, g, k,
            pair_diagram(c, a, b), pair_legs(f, g), legs, legs(true), legs(false))
    }
}

// Commented out: see the note above; pair_limit_is_product_gen carries the
// statement with the product data supplied as hypotheses.
//
// /// Conversely, a limit of the pair diagram is a product: the two legs of the
// /// limit cone are the projections.
// theorem pair_limit_is_product[O, M](c: Category[O, M], a: O, b: O, p: O, legs: Bool -> M) {
//     is_limit(c, pair_diagram(c, a, b), p, legs) implies
//     is_product(c, a, b, p, legs(true), legs(false))
// } by {
//     if is_limit(c, pair_diagram(c, a, b), p, legs) {
//         pair_limit_is_product_gen(c, a, b, p, pair_diagram(c, a, b), legs, legs(true), legs(false))
//     }
// }
