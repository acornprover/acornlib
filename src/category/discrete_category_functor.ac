from category.category import Category, category_identity_dst, category_identity_src, category_id_left
from category.discrete_category import discrete_category, discrete_category_src, discrete_category_dst,
    discrete_category_identity, discrete_category_compose, discrete_compose
from category.functor import Functor, is_functor, functor_src_axiom, functor_dst_axiom,
    functor_identity_axiom, functor_compose_axiom, functor_new_src_cat,
    functor_new_dst_cat, functor_new_obj_map, functor_new_mor_map
from data.basic.functions import identity_fn

/// The morphism map of the functor freely determined by a function out of a
/// discrete category: every source morphism is an identity, so it is sent to the
/// identity at the image object.
define discrete_functor_mor_map[O, O2, M2](d: Category[O2, M2], f: O -> O2, x: O) -> M2 {
    d.identity(f(x))
}

/// The source axiom for the functor induced by a map out of a discrete category.
theorem discrete_functor_src_axiom[O, O2, M2](
    anchor: O, d: Category[O2, M2], f: O -> O2
) {
    functor_src_axiom(discrete_category(anchor), d, f, discrete_functor_mor_map(d, f))
} by {
    let c = discrete_category(anchor)
    discrete_category_src(anchor)
    forall(x: O) {
        category_identity_src(d, f(x))
        d.src(discrete_functor_mor_map(d, f, x)) = f(c.src(x))
    }
}

/// The target axiom for the functor induced by a map out of a discrete category.
theorem discrete_functor_dst_axiom[O, O2, M2](
    anchor: O, d: Category[O2, M2], f: O -> O2
) {
    functor_dst_axiom(discrete_category(anchor), d, f, discrete_functor_mor_map(d, f))
} by {
    let c = discrete_category(anchor)
    discrete_category_dst(anchor)
    forall(x: O) {
        category_identity_dst(d, f(x))
        d.dst(discrete_functor_mor_map(d, f, x)) = f(c.dst(x))
    }
}

/// The identity axiom for the functor induced by a map out of a discrete category.
theorem discrete_functor_identity_axiom[O, O2, M2](
    anchor: O, d: Category[O2, M2], f: O -> O2
) {
    functor_identity_axiom(discrete_category(anchor), d, f, discrete_functor_mor_map(d, f))
} by {
    let c = discrete_category(anchor)
    discrete_category_identity(anchor)
    forall(x: O) {
        discrete_functor_mor_map(d, f, c.identity(x)) = d.identity(f(x))
    }
}

/// The composition axiom for the functor induced by a map out of a discrete category.
theorem discrete_functor_compose_axiom[O, O2, M2](
    anchor: O, d: Category[O2, M2], f: O -> O2
) {
    functor_compose_axiom(discrete_category(anchor), d, f, discrete_functor_mor_map(d, f))
} by {
    let c = discrete_category(anchor)
    discrete_category_src(anchor)
    discrete_category_dst(anchor)
    discrete_category_compose(anchor)
    forall(p: O, q: O) {
        if c.src(p) = c.dst(q) {


            discrete_functor_mor_map(d, f, c.compose(p, q)) = d.identity(f(p))

            category_identity_dst(d, f(p))
            category_id_left(d, d.identity(f(p)))

            discrete_functor_mor_map(d, f, c.compose(p, q)) =
                d.compose(discrete_functor_mor_map(d, f, p), discrete_functor_mor_map(d, f, q))
        }
    }
    functor_compose_axiom(c, d, f, discrete_functor_mor_map(d, f))
}

/// A map from the objects of a discrete category into any category determines a functor.
theorem discrete_functor_is_functor[O, O2, M2](
    anchor: O, d: Category[O2, M2], f: O -> O2
) {
    is_functor(discrete_category(anchor), d, f, discrete_functor_mor_map(d, f))
} by {
    discrete_functor_src_axiom(anchor, d, f)
    discrete_functor_dst_axiom(anchor, d, f)
    discrete_functor_identity_axiom(anchor, d, f)
    discrete_functor_compose_axiom(anchor, d, f)
}

/// The bundled functor from a discrete category determined by an object map.
let discrete_functor[O, O2, M2](anchor: O, d: Category[O2, M2], f: O -> O2) -> result: Functor[O, O, O2, M2] satisfy {
    Functor[O, O, O2, M2].new(discrete_category(anchor), d, f, discrete_functor_mor_map(d, f)) =
        Option.some(result)
} by {
    discrete_functor_is_functor(anchor, d, f)
}

/// The bundled discrete functor has the expected source category.
theorem discrete_functor_src_cat[O, O2, M2](anchor: O, d: Category[O2, M2], f: O -> O2) {
    discrete_functor(anchor, d, f).src_cat = discrete_category(anchor)
} by {
    let h = discrete_functor(anchor, d, f)
    Functor[O, O, O2, M2].new(discrete_category(anchor), d, f, discrete_functor_mor_map(d, f)) =
        Option.some(h)
    functor_new_src_cat(discrete_category(anchor), d, f, discrete_functor_mor_map(d, f), h)
}

/// The bundled discrete functor has the expected target category.
theorem discrete_functor_dst_cat[O, O2, M2](anchor: O, d: Category[O2, M2], f: O -> O2) {
    discrete_functor(anchor, d, f).dst_cat = d
} by {
    let h = discrete_functor(anchor, d, f)
    Functor[O, O, O2, M2].new(discrete_category(anchor), d, f, discrete_functor_mor_map(d, f)) =
        Option.some(h)
    functor_new_dst_cat(discrete_category(anchor), d, f, discrete_functor_mor_map(d, f), h)
}

/// The bundled discrete functor has the given object map.
theorem discrete_functor_obj_map[O, O2, M2](anchor: O, d: Category[O2, M2], f: O -> O2) {
    discrete_functor(anchor, d, f).obj_map = f
} by {
    let h = discrete_functor(anchor, d, f)
    Functor[O, O, O2, M2].new(discrete_category(anchor), d, f, discrete_functor_mor_map(d, f)) =
        Option.some(h)
    functor_new_obj_map(discrete_category(anchor), d, f, discrete_functor_mor_map(d, f), h)
}

/// The bundled discrete functor has the identity-valued morphism map.
theorem discrete_functor_mor_map_eq[O, O2, M2](anchor: O, d: Category[O2, M2], f: O -> O2) {
    discrete_functor(anchor, d, f).mor_map = discrete_functor_mor_map(d, f)
} by {
    let h = discrete_functor(anchor, d, f)
    Functor[O, O, O2, M2].new(discrete_category(anchor), d, f, discrete_functor_mor_map(d, f)) =
        Option.some(h)
    functor_new_mor_map(discrete_category(anchor), d, f, discrete_functor_mor_map(d, f), h)
}

/// On each morphism of the discrete category, the bundled functor returns the
/// identity morphism at the image object.
theorem discrete_functor_mor_map_apply[O, O2, M2](
    anchor: O, d: Category[O2, M2], f: O -> O2, x: O
) {
    discrete_functor(anchor, d, f).mor_map(x) = d.identity(f(x))
} by {
    discrete_functor_mor_map_eq(anchor, d, f)
}

/// On each object of the discrete category, the bundled functor evaluates by the
/// original object map.
theorem discrete_functor_obj_map_apply[O, O2, M2](
    anchor: O, d: Category[O2, M2], f: O -> O2, x: O
) {
    discrete_functor(anchor, d, f).obj_map(x) = f(x)
} by {
    discrete_functor_obj_map(anchor, d, f)
}

/// The identity object map out of a discrete category gives the identity-on-data functor.
theorem discrete_functor_identity_obj_map[O](anchor: O) {
    discrete_functor(anchor, discrete_category(anchor), identity_fn[O]).obj_map = identity_fn[O]
} by {
    discrete_functor_obj_map(anchor, discrete_category(anchor), identity_fn[O])
}

/// The identity object map out of a discrete category sends every morphism to
/// the corresponding identity in the discrete target.
theorem discrete_functor_identity_mor_map_apply[O](anchor: O, x: O) {
    discrete_functor(anchor, discrete_category(anchor), identity_fn[O]).mor_map(x) = x
} by {
    discrete_functor_mor_map_apply(anchor, discrete_category(anchor), identity_fn[O], x)
    discrete_category_identity(anchor)
}
