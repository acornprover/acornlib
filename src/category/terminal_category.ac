/// The terminal category: a category with one object and one morphism.

from category.category import Category, category_ext
from category.discrete_category import discrete_category, discrete_category_src,
    discrete_category_dst, discrete_category_identity
from data.basic.functions import function_extensionality, identity_fn
from category.functor import Functor, constant_functor, constant_functor_src_cat,
    constant_functor_dst_cat, constant_functor_obj_eq, constant_functor_mor_eq,
    functor_ext
from category.opposite_category import opposite_category, opposite_category_src,
    opposite_category_dst, opposite_category_identity, opposite_category_compose,
    opposite_compose

/// A type with exactly one inhabitant.
inductive UnitType {
    /// The unique inhabitant.
    point
}

/// Every value of `UnitType` equals `UnitType.point`.
theorem unit_type_unique(x: UnitType) {
    x = UnitType.point
} by {
    match x {
        UnitType.point {
            x = UnitType.point
        }
    }
}

/// Any two values of `UnitType` are equal.
theorem unit_type_eq(x: UnitType, y: UnitType) {
    x = y
} by {
    unit_type_unique(x)
    unit_type_unique(y)
}

/// The terminal category, with one object and one morphism.
let terminal_category: Category[UnitType, UnitType] = discrete_category(UnitType.point)

/// The source map of the terminal category is the identity function.
theorem terminal_category_src {
    terminal_category.src = identity_fn[UnitType]
} by {
    discrete_category_src(UnitType.point)
}

/// The target map of the terminal category is the identity function.
theorem terminal_category_dst {
    terminal_category.dst = identity_fn[UnitType]
} by {
    discrete_category_dst(UnitType.point)
}

/// The identity-assigning map of the terminal category is the identity function.
theorem terminal_category_identity {
    terminal_category.identity = identity_fn[UnitType]
} by {
    discrete_category_identity(UnitType.point)
}

/// Every object in the terminal category equals `UnitType.point`.
theorem terminal_category_object_unique(x: UnitType) {
    x = UnitType.point
} by {
    unit_type_unique(x)
}

/// Every morphism in the terminal category equals `UnitType.point`.
theorem terminal_category_morphism_unique(f: UnitType) {
    f = UnitType.point
} by {
    unit_type_unique(f)
}

/// Any two morphisms in the terminal category are equal.
theorem terminal_category_morphism_eq(f: UnitType, g: UnitType) {
    f = g
} by {
    unit_type_eq(f, g)
}

/// The opposite of the terminal category is itself.
theorem opposite_terminal_category {
    opposite_category(terminal_category) = terminal_category
} by {
    let d = opposite_category(terminal_category)
    opposite_category_src(terminal_category)
    opposite_category_dst(terminal_category)
    opposite_category_identity(terminal_category)
    opposite_category_compose(terminal_category)
    terminal_category_src
    terminal_category_dst
    terminal_category_identity
    forall(m: UnitType) {
        d.src(m) = terminal_category.src(m)
    }
    forall(m: UnitType) {
        d.dst(m) = terminal_category.dst(m)
    }
    forall(x: UnitType) {
        d.identity(x) = terminal_category.identity(x)
    }
    forall(f: UnitType, g: UnitType) {
        d.compose(f, g) = opposite_compose(terminal_category, f, g)
        opposite_compose(terminal_category, f, g) = terminal_category.compose(g, f)
        unit_type_eq(terminal_category.compose(g, f), terminal_category.compose(f, g))
        d.compose(f, g) = terminal_category.compose(f, g)
    }
    category_ext(d, terminal_category)
}

/// Every functor into the terminal category equals the constant functor at the unique point.
theorem unique_functor_to_terminal_category[O, M](f: Functor[O, M, UnitType, UnitType]) {
    f.dst_cat = terminal_category implies
        f = constant_functor(f.src_cat, terminal_category, UnitType.point)
} by {
    if f.dst_cat = terminal_category {
        let g = constant_functor(f.src_cat, terminal_category, UnitType.point)
        constant_functor_src_cat(f.src_cat, terminal_category, UnitType.point)
        g.src_cat = f.src_cat
        constant_functor_dst_cat(f.src_cat, terminal_category, UnitType.point)
        g.dst_cat = terminal_category
        g.dst_cat = f.dst_cat

        forall(x: O) {
            unit_type_unique(f.obj_map(x))
            f.obj_map(x) = UnitType.point
            constant_functor_obj_eq(f.src_cat, terminal_category, UnitType.point, x)
            g.obj_map(x) = UnitType.point
            f.obj_map(x) = g.obj_map(x)
        }
        function_extensionality(f.obj_map, g.obj_map)
        f.obj_map = g.obj_map

        forall(m: M) {
            unit_type_unique(f.mor_map(m))
            f.mor_map(m) = UnitType.point
            constant_functor_mor_eq(f.src_cat, terminal_category, UnitType.point, m)
            g.mor_map(m) = UnitType.point
            f.mor_map(m) = g.mor_map(m)
        }
        function_extensionality(f.mor_map, g.mor_map)
        f.mor_map = g.mor_map

        f.src_cat = g.src_cat
        functor_ext(f, g)
        f = g
        f = constant_functor(f.src_cat, terminal_category, UnitType.point)
    }
}

/// The terminal category has a unique endofunctor.
theorem terminal_category_endofunctor_unique(f: Functor[UnitType, UnitType, UnitType, UnitType]) {
    f.src_cat = terminal_category and f.dst_cat = terminal_category
        implies f = constant_functor(terminal_category, terminal_category, UnitType.point)
} by {
    if f.src_cat = terminal_category and f.dst_cat = terminal_category {
        unique_functor_to_terminal_category(f)
        f = constant_functor(f.src_cat, terminal_category, UnitType.point)
        f.src_cat = terminal_category
        f = constant_functor(terminal_category, terminal_category, UnitType.point)
    }
}

/// Any functor into the terminal category sends every object to the unique object.
theorem functor_to_terminal_obj_map_unique[O, M](
    f: Functor[O, M, UnitType, UnitType], x: O
) {
    f.dst_cat = terminal_category implies f.obj_map(x) = UnitType.point
} by {
    if f.dst_cat = terminal_category {
        unit_type_unique(f.obj_map(x))
    }
}

/// Any functor into the terminal category sends every morphism to the unique morphism.
theorem functor_to_terminal_mor_map_unique[O, M](
    f: Functor[O, M, UnitType, UnitType], m: M
) {
    f.dst_cat = terminal_category implies f.mor_map(m) = UnitType.point
} by {
    if f.dst_cat = terminal_category {
        unit_type_unique(f.mor_map(m))
    }
}
