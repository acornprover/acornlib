/// The Yoneda lemma: natural transformations out of a covariant hom functor
/// Hom(a, -) correspond exactly to elements of the target functor's value at `a`.
///
/// The library's functor machinery targets categories with a fixed object type,
/// and there is no type of all types, so the category of sets (objects = types,
/// morphisms = functions) cannot be constructed and hom objects cannot be made
/// into types.  Instead a set-valued functor is represented directly:
/// each object `b` is assigned a predicate on morphisms (the "elements" of the
/// set at `b`), and each morphism acts on elements.  The Yoneda correspondence
/// below is then the element-level statement: evaluating a natural
/// transformation at the identity of `a` yields an element of F(a), and every
/// element of F(a) yields a natural transformation `b -> (f: a -> b) -> F(f)(x)`,
/// and the two constructions are inverse to each other.

from category.category import Category, category_identity_src, category_identity_dst,
    category_id_left, category_id_right, category_compose_src, category_compose_dst,
    category_compose_assoc

// ---------------------------------------------------------------------------
// Hom-sets and the action of the covariant hom functor Hom(a, -)
// ---------------------------------------------------------------------------

/// The hom-set Hom(a, b): the predicate holding exactly the morphisms from `a` to `b`.
define hom_set[O, M](c: Category[O, M], a: O, b: O, f: M) -> Bool {
    c.src(f) = a and c.dst(f) = b
}

/// The action of Hom(a, -) on a morphism `g`: precomposition, sending
/// `f: a -> b` to `g ∘ f: a -> c` when `g: b -> c`.
define hom_map[O, M](c: Category[O, M], g: M, f: M) -> M {
    c.compose(g, f)
}

/// Precomposition by `g: b -> d` sends Hom(a, b) into Hom(a, d).
theorem hom_map_membership[O, M](c: Category[O, M], a: O, b: O, d: O, g: M, f: M) {
    hom_set(c, a, b, f) and c.src(g) = b and c.dst(g) = d implies
        hom_set(c, a, d, hom_map(c, g, f))
} by {
    if hom_set(c, a, b, f) and c.src(g) = b and c.dst(g) = d {
        hom_set(c, a, b, f) = (c.src(f) = a and c.dst(f) = b)
        c.src(f) = a
        c.dst(f) = b
        c.src(g) = b
        c.dst(g) = d
        c.src(g) = c.dst(f)
        category_compose_src(c, g, f)
        c.src(c.compose(g, f)) = c.src(f)
        c.src(c.compose(g, f)) = a
        category_compose_dst(c, g, f)
        c.dst(c.compose(g, f)) = c.dst(g)
        c.dst(c.compose(g, f)) = d
        hom_map(c, g, f) = c.compose(g, f)
        hom_set(c, a, d, hom_map(c, g, f))
    }
}

/// The action of Hom(a, -) preserves composition: (g ∘ h) ∘ f = g ∘ (h ∘ f).
theorem hom_map_compose[O, M](c: Category[O, M], g: M, h: M, f: M) {
    c.src(g) = c.dst(h) and c.src(h) = c.dst(f) implies
        hom_map(c, g, hom_map(c, h, f)) = hom_map(c, c.compose(g, h), f)
} by {
    if c.src(g) = c.dst(h) and c.src(h) = c.dst(f) {
        hom_map(c, h, f) = c.compose(h, f)
        hom_map(c, g, hom_map(c, h, f)) = c.compose(g, c.compose(h, f))
        hom_map(c, c.compose(g, h), f) = c.compose(c.compose(g, h), f)
        category_compose_assoc(c, g, h, f)
        c.compose(c.compose(g, h), f) = c.compose(g, c.compose(h, f))
        hom_map(c, g, hom_map(c, h, f)) = hom_map(c, c.compose(g, h), f)
    }
}

/// The action of Hom(a, -) preserves identities: id_b ∘ f = f for f: a -> b.
theorem hom_map_identity[O, M](c: Category[O, M], b: O, f: M) {
    c.dst(f) = b implies hom_map(c, c.identity(b), f) = f
} by {
    if c.dst(f) = b {
        category_id_left(c, f)
        c.compose(c.identity(c.dst(f)), f) = f
        hom_map(c, c.identity(b), f) = c.compose(c.identity(b), f)
        c.compose(c.identity(b), f) = c.compose(c.identity(c.dst(f)), f)
        hom_map(c, c.identity(b), f) = f
    }
}

// ---------------------------------------------------------------------------
// Set-valued functors
// ---------------------------------------------------------------------------

/// True if `mor_map` sends elements of F(src(f)) into F(dst(f)) for every morphism `f`.
define set_valued_respects_axiom[O, M](
    c: Category[O, M], obj_map: O -> (M -> Bool), mor_map: M -> (M -> M)
) -> Bool {
    forall(f: M, x: M) {
        obj_map(c.src(f))(x) implies obj_map(c.dst(f))(mor_map(f)(x))
    }
}

/// True if `mor_map` preserves identities: F(id_x) fixes every element of F(x).
define set_valued_identity_axiom[O, M](
    c: Category[O, M], obj_map: O -> (M -> Bool), mor_map: M -> (M -> M)
) -> Bool {
    forall(x: O, y: M) {
        obj_map(x)(y) implies mor_map(c.identity(x))(y) = y
    }
}

/// True if `mor_map` preserves composition on elements: F(f)(F(g)(y)) = F(f ∘ g)(y).
define set_valued_compose_axiom[O, M](
    c: Category[O, M], obj_map: O -> (M -> Bool), mor_map: M -> (M -> M)
) -> Bool {
    forall(f: M, g: M, y: M) {
        c.src(f) = c.dst(g) and obj_map(c.src(g))(y) implies
            mor_map(f)(mor_map(g)(y)) = mor_map(c.compose(f, g))(y)
    }
}

/// True if `obj_map` and `mor_map` form a set-valued functor on `c`.
define is_set_valued_functor[O, M](
    c: Category[O, M], obj_map: O -> (M -> Bool), mor_map: M -> (M -> M)
) -> Bool {
    set_valued_respects_axiom(c, obj_map, mor_map)
    and set_valued_identity_axiom(c, obj_map, mor_map)
    and set_valued_compose_axiom(c, obj_map, mor_map)
}

/// A set-valued functor on a category: a family of sets of morphisms together
/// with an action on elements, preserving identities and composition.
structure SetValuedFunctor[O, M] {
    /// The underlying category.
    category: Category[O, M]

    /// The family of sets: `obj_map(x)` is the set of elements at `x`.
    obj_map: O -> (M -> Bool)

    /// The action on morphisms: `mor_map(f)` sends elements of F(src(f)) into F(dst(f)).
    mor_map: M -> (M -> M)
} constraint {
    is_set_valued_functor(category, obj_map, mor_map)
}

/// Construction of a set-valued functor remembers its category.
theorem set_valued_new_category[O, M](
    c: Category[O, M], obj_map: O -> (M -> Bool), mor_map: M -> (M -> M),
    f: SetValuedFunctor[O, M]
) {
    SetValuedFunctor[O, M].new(c, obj_map, mor_map) = Option.some(f) implies f.category = c
}

/// Construction of a set-valued functor remembers its object map.
theorem set_valued_new_obj_map[O, M](
    c: Category[O, M], obj_map: O -> (M -> Bool), mor_map: M -> (M -> M),
    f: SetValuedFunctor[O, M]
) {
    SetValuedFunctor[O, M].new(c, obj_map, mor_map) = Option.some(f) implies f.obj_map = obj_map
}

/// Construction of a set-valued functor remembers its morphism map.
theorem set_valued_new_mor_map[O, M](
    c: Category[O, M], obj_map: O -> (M -> Bool), mor_map: M -> (M -> M),
    f: SetValuedFunctor[O, M]
) {
    SetValuedFunctor[O, M].new(c, obj_map, mor_map) = Option.some(f) implies f.mor_map = mor_map
}

/// A set-valued functor's data forms a set-valued functor.
theorem set_valued_is_functor[O, M](f: SetValuedFunctor[O, M]) {
    is_set_valued_functor(f.category, f.obj_map, f.mor_map)
} by {
}

/// A set-valued functor sends elements of F(src(f)) into F(dst(f)).
theorem set_valued_respects[O, M](f: SetValuedFunctor[O, M], g: M, y: M) {
    f.obj_map(f.category.src(g))(y) implies
        f.obj_map(f.category.dst(g))(f.mor_map(g)(y))
} by {
    set_valued_is_functor(f)
    is_set_valued_functor(f.category, f.obj_map, f.mor_map) =
        (set_valued_respects_axiom(f.category, f.obj_map, f.mor_map)
        and set_valued_identity_axiom(f.category, f.obj_map, f.mor_map)
        and set_valued_compose_axiom(f.category, f.obj_map, f.mor_map))
    set_valued_respects_axiom(f.category, f.obj_map, f.mor_map) = forall(h: M, x: M) {
        f.obj_map(f.category.src(h))(x) implies
            f.obj_map(f.category.dst(h))(f.mor_map(h)(x))
    }
}

/// A set-valued functor fixes the elements of each set at the identity.
theorem set_valued_identity[O, M](f: SetValuedFunctor[O, M], x: O, y: M) {
    f.obj_map(x)(y) implies f.mor_map(f.category.identity(x))(y) = y
} by {
    set_valued_is_functor(f)
    is_set_valued_functor(f.category, f.obj_map, f.mor_map) =
        (set_valued_respects_axiom(f.category, f.obj_map, f.mor_map)
        and set_valued_identity_axiom(f.category, f.obj_map, f.mor_map)
        and set_valued_compose_axiom(f.category, f.obj_map, f.mor_map))
    set_valued_identity_axiom(f.category, f.obj_map, f.mor_map) = forall(x0: O, y0: M) {
        f.obj_map(x0)(y0) implies f.mor_map(f.category.identity(x0))(y0) = y0
    }
}

/// A set-valued functor preserves composition on elements.
theorem set_valued_compose[O, M](f: SetValuedFunctor[O, M], p: M, q: M, y: M) {
    f.category.src(p) = f.category.dst(q) and f.obj_map(f.category.src(q))(y) implies
        f.mor_map(p)(f.mor_map(q)(y)) = f.mor_map(f.category.compose(p, q))(y)
} by {
    set_valued_is_functor(f)
    is_set_valued_functor(f.category, f.obj_map, f.mor_map) =
        (set_valued_respects_axiom(f.category, f.obj_map, f.mor_map)
        and set_valued_identity_axiom(f.category, f.obj_map, f.mor_map)
        and set_valued_compose_axiom(f.category, f.obj_map, f.mor_map))
    set_valued_compose_axiom(f.category, f.obj_map, f.mor_map) = forall(h: M, k: M, z: M) {
        f.category.src(h) = f.category.dst(k) and f.obj_map(f.category.src(k))(z) implies
            f.mor_map(h)(f.mor_map(k)(z)) = f.mor_map(f.category.compose(h, k))(z)
    }
}

// ---------------------------------------------------------------------------
// Natural transformations from Hom(a, -) to a set-valued functor
// ---------------------------------------------------------------------------

/// True if `component` sends Hom(a, b) into F(b) at every object `b`.
define yoneda_endpoints_axiom[O, M](
    c: Category[O, M], a: O, f: SetValuedFunctor[O, M], component: O -> (M -> M)
) -> Bool {
    forall(b: O, x: M) {
        hom_set(c, a, b, x) implies f.obj_map(b)(component(b)(x))
    }
}

/// True if the naturality square commutes: for `g: b -> c` and `x: a -> b`,
/// component(c)(g ∘ x) = F(g)(component(b)(x)).
define yoneda_naturality_axiom[O, M](
    c: Category[O, M], a: O, f: SetValuedFunctor[O, M], component: O -> (M -> M)
) -> Bool {
    forall(g: M, x: M) {
        hom_set(c, a, c.src(g), x) implies
            component(c.dst(g))(c.compose(g, x)) = f.mor_map(g)(component(c.src(g))(x))
    }
}

/// True if `component` is a natural transformation from Hom(a, -) to `f`.
define is_yoneda_nat_trans[O, M](
    c: Category[O, M], a: O, f: SetValuedFunctor[O, M], component: O -> (M -> M)
) -> Bool {
    f.category = c
    and yoneda_endpoints_axiom(c, a, f, component)
    and yoneda_naturality_axiom(c, a, f, component)
}

/// A natural transformation from the covariant hom functor Hom(a, -) to a set-valued functor.
structure YonedaNatTrans[O, M] {
    /// The object parameterizing the hom functor.
    object: O

    /// The target set-valued functor.
    target: SetValuedFunctor[O, M]

    /// The component map: `component(b)` sends Hom(a, b) into F(b).
    component: O -> (M -> M)
} constraint {
    is_yoneda_nat_trans(target.category, object, target, component)
}

/// Construction of a natural transformation remembers its object.
theorem yoneda_new_object[O, M](
    a: O, f: SetValuedFunctor[O, M], component: O -> (M -> M),
    n: YonedaNatTrans[O, M]
) {
    YonedaNatTrans[O, M].new(a, f, component) = Option.some(n) implies n.object = a
}

/// Construction of a natural transformation remembers its target functor.
theorem yoneda_new_target[O, M](
    a: O, f: SetValuedFunctor[O, M], component: O -> (M -> M),
    n: YonedaNatTrans[O, M]
) {
    YonedaNatTrans[O, M].new(a, f, component) = Option.some(n) implies n.target = f
}

/// Construction of a natural transformation remembers its component map.
theorem yoneda_new_component[O, M](
    a: O, f: SetValuedFunctor[O, M], component: O -> (M -> M),
    n: YonedaNatTrans[O, M]
) {
    YonedaNatTrans[O, M].new(a, f, component) = Option.some(n) implies n.component = component
}

/// A natural transformation's data is a natural transformation.
theorem yoneda_is_nat_trans[O, M](n: YonedaNatTrans[O, M]) {
    is_yoneda_nat_trans(n.target.category, n.object, n.target, n.component)
} by {
}

/// The component map sends Hom(a, b) into F(b).
theorem yoneda_endpoints[O, M](n: YonedaNatTrans[O, M], b: O, x: M) {
    hom_set(n.target.category, n.object, b, x) implies
        n.target.obj_map(b)(n.component(b)(x))
} by {
    yoneda_is_nat_trans(n)
    is_yoneda_nat_trans(n.target.category, n.object, n.target, n.component) =
        (n.target.category = n.target.category
        and yoneda_endpoints_axiom(n.target.category, n.object, n.target, n.component)
        and yoneda_naturality_axiom(n.target.category, n.object, n.target, n.component))
    yoneda_endpoints_axiom(n.target.category, n.object, n.target, n.component) = forall(b0: O, y: M) {
        hom_set(n.target.category, n.object, b0, y) implies
            n.target.obj_map(b0)(n.component(b0)(y))
    }
}

/// The naturality square commutes for every morphism.
theorem yoneda_naturality[O, M](n: YonedaNatTrans[O, M], g: M, x: M) {
    hom_set(n.target.category, n.object, n.target.category.src(g), x) implies
        n.component(n.target.category.dst(g))(n.target.category.compose(g, x))
            = n.target.mor_map(g)(n.component(n.target.category.src(g))(x))
} by {
    yoneda_is_nat_trans(n)
    is_yoneda_nat_trans(n.target.category, n.object, n.target, n.component) =
        (n.target.category = n.target.category
        and yoneda_endpoints_axiom(n.target.category, n.object, n.target, n.component)
        and yoneda_naturality_axiom(n.target.category, n.object, n.target, n.component))
    yoneda_naturality_axiom(n.target.category, n.object, n.target, n.component) = forall(h: M, y: M) {
        hom_set(n.target.category, n.object, n.target.category.src(h), y) implies
            n.component(n.target.category.dst(h))(n.target.category.compose(h, y))
                = n.target.mor_map(h)(n.component(n.target.category.src(h))(y))
    }
}

// ---------------------------------------------------------------------------
// The Yoneda correspondence
// ---------------------------------------------------------------------------

/// The Yoneda element of a natural transformation: the value of the component
/// at `a` on the identity morphism of `a`.
define yoneda_eval[O, M](n: YonedaNatTrans[O, M]) -> M {
    n.component(n.object)(n.target.category.identity(n.object))
}

/// The Yoneda element of a natural transformation lies in F(a).
theorem yoneda_eval_membership[O, M](n: YonedaNatTrans[O, M]) {
    n.target.obj_map(n.object)(yoneda_eval(n))
} by {
    category_identity_src(n.target.category, n.object)
    category_identity_dst(n.target.category, n.object)
    hom_set(n.target.category, n.object, n.object,
        n.target.category.identity(n.object))
    yoneda_endpoints(n, n.object, n.target.category.identity(n.object))
    n.target.obj_map(n.object)(n.component(n.object)(n.target.category.identity(n.object)))
    yoneda_eval(n) = n.component(n.object)(n.target.category.identity(n.object))
    n.target.obj_map(n.object)(yoneda_eval(n))
}

/// The component at `b` of the Yoneda transform of an element `x` of F(a):
/// the map sending `f: a -> b` to F(f)(x).
define yoneda_component[O, M](f: SetValuedFunctor[O, M], x: M, b: O) -> M -> M {
    function(g: M) { f.mor_map(g)(x) }
}

/// The component map of the Yoneda transform of an element `x` of F(a).
define yoneda_transform_component[O, M](f: SetValuedFunctor[O, M], x: M) -> O -> (M -> M) {
    function(b: O) { yoneda_component(f, x, b) }
}

/// The Yoneda transform of an element of F(a) has the right endpoints.
theorem yoneda_transform_endpoints[O, M](
    c: Category[O, M], a: O, f: SetValuedFunctor[O, M], x: M
) {
    f.category = c and f.obj_map(a)(x) implies
        yoneda_endpoints_axiom(c, a, f, yoneda_transform_component(f, x))
} by {
    if f.category = c and f.obj_map(a)(x) {
        forall(b: O, g: M) {
            if hom_set(c, a, b, g) {
                hom_set(c, a, b, g) = (c.src(g) = a and c.dst(g) = b)
                c.src(g) = a
                c.dst(g) = b
                f.category = c
                f.category.src(g) = c.src(g)
                f.category.dst(g) = c.dst(g)
                f.obj_map(f.category.src(g))(x) = f.obj_map(a)(x)
                f.obj_map(f.category.src(g))(x)
                set_valued_respects(f, g, x)
                f.obj_map(f.category.dst(g))(f.mor_map(g)(x))
                f.obj_map(b)(f.mor_map(g)(x))
                yoneda_transform_component(f, x)(b) = yoneda_component(f, x, b)
                yoneda_component(f, x, b) = function(h: M) { f.mor_map(h)(x) }
                yoneda_component(f, x, b)(g) = f.mor_map(g)(x)
                yoneda_transform_component(f, x)(b)(g) = f.mor_map(g)(x)
                f.obj_map(b)(yoneda_transform_component(f, x)(b)(g))
            }
        }
    }
}

/// The Yoneda transform of an element of F(a) is natural.
theorem yoneda_transform_naturality[O, M](
    c: Category[O, M], a: O, f: SetValuedFunctor[O, M], x: M
) {
    f.category = c and f.obj_map(a)(x) implies
        yoneda_naturality_axiom(c, a, f, yoneda_transform_component(f, x))
} by {
    if f.category = c and f.obj_map(a)(x) {
        forall(g: M, h: M) {
            if hom_set(c, a, c.src(g), h) {
                hom_set(c, a, c.src(g), h) = (c.src(h) = a and c.dst(h) = c.src(g))
                c.src(h) = a
                c.dst(h) = c.src(g)
                c.src(g) = c.dst(h)
                f.category = c
                f.category.src(h) = c.src(h)
                f.obj_map(f.category.src(h))(x) = f.obj_map(a)(x)
                f.obj_map(f.category.src(h))(x)
                set_valued_compose(f, g, h, x)
                f.mor_map(g)(f.mor_map(h)(x)) = f.mor_map(f.category.compose(g, h))(x)
                f.category.compose(g, h) = c.compose(g, h)
                f.mor_map(g)(f.mor_map(h)(x)) = f.mor_map(c.compose(g, h))(x)
                yoneda_transform_component(f, x)(c.dst(g)) = yoneda_component(f, x, c.dst(g))
                yoneda_component(f, x, c.dst(g)) = function(k: M) { f.mor_map(k)(x) }
                yoneda_component(f, x, c.dst(g))(c.compose(g, h)) = f.mor_map(c.compose(g, h))(x)
                yoneda_transform_component(f, x)(c.dst(g))(c.compose(g, h)) = f.mor_map(c.compose(g, h))(x)
                yoneda_transform_component(f, x)(c.src(g)) = yoneda_component(f, x, c.src(g))
                yoneda_component(f, x, c.src(g)) = function(k: M) { f.mor_map(k)(x) }
                yoneda_component(f, x, c.src(g))(h) = f.mor_map(h)(x)
                yoneda_transform_component(f, x)(c.src(g))(h) = f.mor_map(h)(x)
                f.mor_map(g)(yoneda_transform_component(f, x)(c.src(g))(h)) = f.mor_map(g)(f.mor_map(h)(x))
                f.mor_map(g)(yoneda_transform_component(f, x)(c.src(g))(h)) = f.mor_map(c.compose(g, h))(x)
                yoneda_transform_component(f, x)(c.dst(g))(c.compose(g, h)) = f.mor_map(g)(yoneda_transform_component(f, x)(c.src(g))(h))
            }
        }
    }
}

/// The Yoneda transform of an element of F(a) is a natural transformation from Hom(a, -) to F.
theorem yoneda_transform_is_nat_trans[O, M](
    c: Category[O, M], a: O, f: SetValuedFunctor[O, M], x: M
) {
    f.category = c and f.obj_map(a)(x) implies
        is_yoneda_nat_trans(c, a, f, yoneda_transform_component(f, x))
} by {
    if f.category = c and f.obj_map(a)(x) {
        yoneda_transform_endpoints(c, a, f, x)
        yoneda_transform_naturality(c, a, f, x)
        is_yoneda_nat_trans(c, a, f, yoneda_transform_component(f, x))
    }
}

/// The Yoneda transform of an element `x` of F(a), packaged as an optional
/// natural transformation: `Option.some(...)` exactly when `x` lies in F(a)
/// and `f` is over `c`.
define yoneda_apply_opt[O, M](c: Category[O, M], a: O, f: SetValuedFunctor[O, M], x: M) -> Option[YonedaNatTrans[O, M]] {
    YonedaNatTrans[O, M].new(a, f, yoneda_transform_component(f, x))
}

/// The Yoneda transform of an element of F(a) is some bundled natural transformation.
theorem yoneda_apply_some[O, M](c: Category[O, M], a: O, f: SetValuedFunctor[O, M], x: M) {
    f.category = c and f.obj_map(a)(x) implies
        exists(n: YonedaNatTrans[O, M]) {
            yoneda_apply_opt(c, a, f, x) = Option.some(n)
        }
} by {
    if f.category = c and f.obj_map(a)(x) {
        yoneda_transform_is_nat_trans(c, a, f, x)
        let n: YonedaNatTrans[O, M] satisfy {
            YonedaNatTrans[O, M].new(a, f, yoneda_transform_component(f, x)) = Option.some(n)
        }
        yoneda_apply_opt(c, a, f, x) = YonedaNatTrans[O, M].new(a, f, yoneda_transform_component(f, x))
        yoneda_apply_opt(c, a, f, x) = Option.some(n)
    }
}

/// A bundled Yoneda transform has the source object as its object.
theorem yoneda_apply_object[O, M](c: Category[O, M], a: O, f: SetValuedFunctor[O, M], x: M, n: YonedaNatTrans[O, M]) {
    yoneda_apply_opt(c, a, f, x) = Option.some(n) implies n.object = a
} by {
    if yoneda_apply_opt(c, a, f, x) = Option.some(n) {
        yoneda_apply_opt(c, a, f, x) = YonedaNatTrans[O, M].new(a, f, yoneda_transform_component(f, x))
        YonedaNatTrans[O, M].new(a, f, yoneda_transform_component(f, x)) = Option.some(n)
        yoneda_new_object(a, f, yoneda_transform_component(f, x), n)
    }
}

/// A bundled Yoneda transform has the target functor as its target.
theorem yoneda_apply_target[O, M](c: Category[O, M], a: O, f: SetValuedFunctor[O, M], x: M, n: YonedaNatTrans[O, M]) {
    yoneda_apply_opt(c, a, f, x) = Option.some(n) implies n.target = f
} by {
    if yoneda_apply_opt(c, a, f, x) = Option.some(n) {
        yoneda_apply_opt(c, a, f, x) = YonedaNatTrans[O, M].new(a, f, yoneda_transform_component(f, x))
        YonedaNatTrans[O, M].new(a, f, yoneda_transform_component(f, x)) = Option.some(n)
        yoneda_new_target(a, f, yoneda_transform_component(f, x), n)
    }
}

/// A bundled Yoneda transform has the pointwise component map.
theorem yoneda_apply_component_eq[O, M](c: Category[O, M], a: O, f: SetValuedFunctor[O, M], x: M, n: YonedaNatTrans[O, M]) {
    yoneda_apply_opt(c, a, f, x) = Option.some(n) implies
        n.component = yoneda_transform_component(f, x)
} by {
    if yoneda_apply_opt(c, a, f, x) = Option.some(n) {
        yoneda_apply_opt(c, a, f, x) = YonedaNatTrans[O, M].new(a, f, yoneda_transform_component(f, x))
        YonedaNatTrans[O, M].new(a, f, yoneda_transform_component(f, x)) = Option.some(n)
        yoneda_new_component(a, f, yoneda_transform_component(f, x), n)
    }
}

/// Round trip, one direction: applying the Yoneda transform to the Yoneda
/// element of a natural transformation recovers the transformation on every
/// hom-set.
theorem yoneda_round_trip_left[O, M](n: YonedaNatTrans[O, M], b: O, g: M) {
    hom_set(n.target.category, n.object, b, g) implies
        yoneda_component(n.target, yoneda_eval(n), b)(g) = n.component(b)(g)
} by {
    if hom_set(n.target.category, n.object, b, g) {
        hom_set(n.target.category, n.object, b, g) =
            (n.target.category.src(g) = n.object and n.target.category.dst(g) = b)
        n.target.category.src(g) = n.object
        n.target.category.dst(g) = b
        category_identity_src(n.target.category, n.object)
        category_identity_dst(n.target.category, n.object)
        hom_set(n.target.category, n.object, n.target.category.src(g),
            n.target.category.identity(n.object))
        yoneda_naturality(n, g, n.target.category.identity(n.object))
        n.component(n.target.category.dst(g))(n.target.category.compose(g, n.target.category.identity(n.object))) = n.target.mor_map(g)(n.component(n.target.category.src(g))(n.target.category.identity(n.object)))
        n.component(b)(n.target.category.compose(g, n.target.category.identity(n.object))) = n.target.mor_map(g)(n.component(n.object)(n.target.category.identity(n.object)))
        category_id_right(n.target.category, g)
        n.target.category.compose(g, n.target.category.identity(n.target.category.src(g))) = g
        n.target.category.compose(g, n.target.category.identity(n.object)) = g
        n.component(b)(g) = n.target.mor_map(g)(n.component(n.object)(n.target.category.identity(n.object)))
        yoneda_eval(n) = n.component(n.object)(n.target.category.identity(n.object))
        n.component(b)(g) = n.target.mor_map(g)(yoneda_eval(n))
        yoneda_component(n.target, yoneda_eval(n), b) = function(h: M) { n.target.mor_map(h)(yoneda_eval(n)) }
        yoneda_component(n.target, yoneda_eval(n), b)(g) = n.target.mor_map(g)(yoneda_eval(n))
        yoneda_component(n.target, yoneda_eval(n), b)(g) = n.component(b)(g)
    }
}

/// The Yoneda element of a bundled Yoneda transform of `x` is F(id_a)(x).
theorem yoneda_eval_apply[O, M](c: Category[O, M], a: O, f: SetValuedFunctor[O, M], x: M, n: YonedaNatTrans[O, M]) {
    f.category = c and f.obj_map(a)(x) and yoneda_apply_opt(c, a, f, x) = Option.some(n) implies
        yoneda_eval(n) = f.mor_map(f.category.identity(a))(x)
} by {
    if f.category = c and f.obj_map(a)(x) and yoneda_apply_opt(c, a, f, x) = Option.some(n) {
        yoneda_apply_opt(c, a, f, x) = YonedaNatTrans[O, M].new(a, f, yoneda_transform_component(f, x))
        YonedaNatTrans[O, M].new(a, f, yoneda_transform_component(f, x)) = Option.some(n)
        yoneda_new_object(a, f, yoneda_transform_component(f, x), n)
        n.object = a
        yoneda_new_target(a, f, yoneda_transform_component(f, x), n)
        n.target = f
        yoneda_new_component(a, f, yoneda_transform_component(f, x), n)
        n.component = yoneda_transform_component(f, x)
        yoneda_eval(n) = n.component(n.object)(n.target.category.identity(n.object))
        n.target.category = f.category
        n.target.category.identity(n.object) = f.category.identity(a)
        n.component(n.object) = yoneda_transform_component(f, x)(a)
        yoneda_eval(n) = yoneda_transform_component(f, x)(a)(f.category.identity(a))
        yoneda_transform_component(f, x)(a) = yoneda_component(f, x, a)
        yoneda_eval(n) = yoneda_component(f, x, a)(f.category.identity(a))
        yoneda_component(f, x, a) = function(h: M) { f.mor_map(h)(x) }
        yoneda_eval(n) = f.mor_map(f.category.identity(a))(x)
    }
}

/// Round trip, the other direction: the Yoneda element of any bundled Yoneda
/// transform of an element `x` of F(a) is `x` itself.
theorem yoneda_round_trip_right[O, M](c: Category[O, M], a: O, f: SetValuedFunctor[O, M], x: M, n: YonedaNatTrans[O, M]) {
    f.category = c and f.obj_map(a)(x) and yoneda_apply_opt(c, a, f, x) = Option.some(n) implies
        yoneda_eval(n) = x
} by {
    if f.category = c and f.obj_map(a)(x) and yoneda_apply_opt(c, a, f, x) = Option.some(n) {
        yoneda_eval_apply(c, a, f, x, n)
        yoneda_eval(n) = f.mor_map(f.category.identity(a))(x)
        set_valued_identity(f, a, x)
        f.mor_map(f.category.identity(a))(x) = x
        yoneda_eval(n) = x
    }
}

/// The Yoneda element map is injective: two natural transformations that agree
/// with the transform of their Yoneda element on every hom-set agree everywhere.
theorem yoneda_lemma_left[O, M](c: Category[O, M], a: O, f: SetValuedFunctor[O, M]) {
    f.category = c implies
        forall(n: YonedaNatTrans[O, M], b: O, g: M) {
            n.target = f and n.object = a and hom_set(c, a, b, g) implies
                yoneda_component(n.target, yoneda_eval(n), b)(g) = n.component(b)(g)
        }
} by {
    if f.category = c {
        forall(n: YonedaNatTrans[O, M], b: O, g: M) {
            if n.target = f and n.object = a and hom_set(c, a, b, g) {
                n.target = f
                n.target.category = f.category
                n.target.category = c
                n.object = a
                hom_set(n.target.category, n.object, b, g)
                yoneda_round_trip_left(n, b, g)
                yoneda_component(n.target, yoneda_eval(n), b)(g) = n.component(b)(g)
            }
        }
    }
}

/// The Yoneda transform is a right inverse of the Yoneda element map.
theorem yoneda_lemma_right[O, M](c: Category[O, M], a: O, f: SetValuedFunctor[O, M]) {
    f.category = c implies
        forall(x: M, n: YonedaNatTrans[O, M]) {
            f.obj_map(a)(x) and yoneda_apply_opt(c, a, f, x) = Option.some(n) implies
                yoneda_eval(n) = x
        }
} by {
    if f.category = c {
        forall(x: M, n: YonedaNatTrans[O, M]) {
            if f.obj_map(a)(x) and yoneda_apply_opt(c, a, f, x) = Option.some(n) {
                yoneda_round_trip_right(c, a, f, x, n)
                yoneda_eval(n) = x
            }
        }
    }
}

/// The Yoneda lemma: the Yoneda element map and the Yoneda transform are
/// inverse to each other, so natural transformations Hom(a, -) => F correspond
/// bijectively to the elements of F(a).
theorem yoneda_lemma[O, M](c: Category[O, M], a: O, f: SetValuedFunctor[O, M]) {
    f.category = c implies
        (forall(n: YonedaNatTrans[O, M], b: O, g: M) {
            n.target = f and n.object = a and hom_set(c, a, b, g) implies
                yoneda_component(n.target, yoneda_eval(n), b)(g) = n.component(b)(g)
        })
        and (forall(x: M, n: YonedaNatTrans[O, M]) {
            f.obj_map(a)(x) and yoneda_apply_opt(c, a, f, x) = Option.some(n) implies
                yoneda_eval(n) = x
        })
} by {
    if f.category = c {
        yoneda_lemma_left(c, a, f)
        forall(n: YonedaNatTrans[O, M], b: O, g: M) {
            n.target = f and n.object = a and hom_set(c, a, b, g) implies yoneda_component(n.target, yoneda_eval(n), b)(g) = n.component(b)(g)
        }
        yoneda_lemma_right(c, a, f)
        forall(x: M, n: YonedaNatTrans[O, M]) {
            f.obj_map(a)(x) and yoneda_apply_opt(c, a, f, x) = Option.some(n) implies yoneda_eval(n) = x
        }
    }
}

// ---------------------------------------------------------------------------
// The covariant hom functor Hom(b, -) is a set-valued functor
// ---------------------------------------------------------------------------

/// The object map of the hom functor Hom(a, -).
define hom_functor_obj[O, M](c: Category[O, M], a: O, b: O) -> M -> Bool {
    function(f: M) { hom_set(c, a, b, f) }
}

/// The morphism map of the hom functor Hom(a, -): precomposition.
define hom_functor_mor[O, M](c: Category[O, M], g: M) -> M -> M {
    function(f: M) { hom_map(c, g, f) }
}

/// The hom functor Hom(a, -) respects the sets.
theorem hom_functor_respects_axiom[O, M](c: Category[O, M], a: O) {
    set_valued_respects_axiom(c, hom_functor_obj(c, a), hom_functor_mor(c))
} by {
    forall(g: M, x: M) {
        if hom_functor_obj(c, a, c.src(g), x) {
            hom_functor_obj(c, a, c.src(g), x) = function(f: M) { hom_set(c, a, c.src(g), f) }(x)
            function(f: M) { hom_set(c, a, c.src(g), f) }(x) = hom_set(c, a, c.src(g), x)
            hom_set(c, a, c.src(g), x)
            hom_map_membership(c, a, c.src(g), c.dst(g), g, x)
            hom_set(c, a, c.dst(g), hom_map(c, g, x))
            hom_functor_obj(c, a, c.dst(g), hom_map(c, g, x)) =
                function(f: M) { hom_set(c, a, c.dst(g), f) }(hom_map(c, g, x))
            function(f: M) { hom_set(c, a, c.dst(g), f) }(hom_map(c, g, x)) = hom_set(c, a, c.dst(g), hom_map(c, g, x))
            hom_functor_obj(c, a, c.dst(g), hom_map(c, g, x))
            hom_functor_mor(c, g) = function(f: M) { hom_map(c, g, f) }
            hom_functor_mor(c, g, x) = hom_map(c, g, x)
            hom_functor_obj(c, a, c.dst(g))(hom_functor_mor(c, g)(x))
        }
    }
}

/// The hom functor Hom(a, -) preserves identities.
theorem hom_functor_identity_axiom[O, M](c: Category[O, M], a: O) {
    set_valued_identity_axiom(c, hom_functor_obj(c, a), hom_functor_mor(c))
} by {
    forall(x: O, y: M) {
        if hom_functor_obj(c, a, x, y) {
            hom_functor_obj(c, a, x, y) = function(f: M) { hom_set(c, a, x, f) }(y)
            function(f: M) { hom_set(c, a, x, f) }(y) = hom_set(c, a, x, y)
            hom_set(c, a, x, y) = (c.src(y) = a and c.dst(y) = x)
            c.dst(y) = x
            hom_map_identity(c, x, y)
            hom_map(c, c.identity(x), y) = y
            hom_functor_mor(c, c.identity(x)) = function(f: M) { hom_map(c, c.identity(x), f) }
            hom_functor_mor(c, c.identity(x), y) = hom_map(c, c.identity(x), y)
            hom_functor_mor(c, c.identity(x))(y) = y
        }
    }
}

/// The hom functor Hom(a, -) preserves composition.
theorem hom_functor_compose_axiom[O, M](c: Category[O, M], a: O) {
    set_valued_compose_axiom(c, hom_functor_obj(c, a), hom_functor_mor(c))
} by {
    forall(f: M, g: M, y: M) {
        if c.src(f) = c.dst(g) and hom_functor_obj(c, a, c.src(g), y) {
            hom_functor_obj(c, a, c.src(g), y) = function(h: M) { hom_set(c, a, c.src(g), h) }(y)
            function(h: M) { hom_set(c, a, c.src(g), h) }(y) = hom_set(c, a, c.src(g), y)
            hom_set(c, a, c.src(g), y) = (c.src(y) = a and c.dst(y) = c.src(g))
            c.dst(y) = c.src(g)
            c.src(g) = c.dst(y)
            hom_map_compose(c, f, g, y)
            hom_map(c, f, hom_map(c, g, y)) = hom_map(c, c.compose(f, g), y)
            hom_functor_mor(c, f) = function(h: M) { hom_map(c, f, h) }
            hom_functor_mor(c, g) = function(h: M) { hom_map(c, g, h) }
            hom_functor_mor(c, c.compose(f, g)) = function(h: M) { hom_map(c, c.compose(f, g), h) }
            hom_functor_mor(c, f, hom_functor_mor(c, g, y)) = hom_map(c, f, hom_map(c, g, y))
            hom_functor_mor(c, c.compose(f, g), y) = hom_map(c, c.compose(f, g), y)
            hom_functor_mor(c, f)(hom_functor_mor(c, g)(y)) = hom_functor_mor(c, c.compose(f, g))(y)
        }
    }
}

/// The hom functor Hom(a, -) is a set-valued functor.
theorem hom_functor_is_set_valued[O, M](c: Category[O, M], a: O) {
    is_set_valued_functor(c, hom_functor_obj(c, a), hom_functor_mor(c))
} by {
    hom_functor_respects_axiom(c, a)
    hom_functor_identity_axiom(c, a)
    hom_functor_compose_axiom(c, a)
}

/// The covariant hom functor Hom(a, -).
let hom_functor[O, M](c: Category[O, M], a: O) -> result: SetValuedFunctor[O, M] satisfy {
    SetValuedFunctor[O, M].new(c, hom_functor_obj(c, a), hom_functor_mor(c)) = Option.some(result)
} by {
    hom_functor_is_set_valued(c, a)
}

/// The hom functor Hom(a, -) has the original category as its underlying category.
theorem hom_functor_category[O, M](c: Category[O, M], a: O) {
    hom_functor(c, a).category = c
} by {
    let f = hom_functor(c, a)
    (SetValuedFunctor[O, M].new(c, hom_functor_obj(c, a), hom_functor_mor(c)) = Option.some(f))
    set_valued_new_category(c, hom_functor_obj(c, a), hom_functor_mor(c), f)
}

/// The hom functor Hom(a, -) has the pointwise object map.
theorem hom_functor_obj_map[O, M](c: Category[O, M], a: O) {
    hom_functor(c, a).obj_map = hom_functor_obj(c, a)
} by {
    let f = hom_functor(c, a)
    (SetValuedFunctor[O, M].new(c, hom_functor_obj(c, a), hom_functor_mor(c)) = Option.some(f))
    set_valued_new_obj_map(c, hom_functor_obj(c, a), hom_functor_mor(c), f)
}

/// The hom functor Hom(a, -) has the pointwise morphism map.
theorem hom_functor_mor_map[O, M](c: Category[O, M], a: O) {
    hom_functor(c, a).mor_map = hom_functor_mor(c)
} by {
    let f = hom_functor(c, a)
    (SetValuedFunctor[O, M].new(c, hom_functor_obj(c, a), hom_functor_mor(c)) = Option.some(f))
    set_valued_new_mor_map(c, hom_functor_obj(c, a), hom_functor_mor(c), f)
}

/// The set of morphisms Hom(a, b) is the object of the hom functor at `b`.
theorem hom_functor_obj_apply[O, M](c: Category[O, M], a: O, b: O, f: M) {
    hom_functor(c, a).obj_map(b)(f) = hom_set(c, a, b, f)
} by {
    hom_functor_obj_map(c, a)
    hom_functor_obj(c, a, b) = function(g: M) { hom_set(c, a, b, g) }
    hom_functor(c, a).obj_map(b)(f) = hom_functor_obj(c, a, b)(f)
    hom_functor_obj(c, a, b)(f) = hom_set(c, a, b, f)
}

/// The action of the hom functor Hom(a, -) on `g` sends `f` to `g ∘ f`.
theorem hom_functor_mor_apply[O, M](c: Category[O, M], a: O, g: M, f: M) {
    hom_functor(c, a).mor_map(g)(f) = hom_map(c, g, f)
} by {
    hom_functor_mor_map(c, a)
    hom_functor_mor(c, g) = function(h: M) { hom_map(c, g, h) }
    hom_functor(c, a).mor_map(g)(f) = hom_functor_mor(c, g)(f)
    hom_functor_mor(c, g)(f) = hom_map(c, g, f)
}
