/// Limits and cones in a category.
///
/// A diagram is a functor from a small category (the shape) into a category.
/// A cone over a diagram `d` is an object `n` together with a morphism
/// `legs(i): n -> d.obj_map(i)` for every object `i` of the shape, such that
/// every triangle `compose(d.mor_map(f), legs(src(f))) = legs(dst(f))` commutes
/// for every morphism `f` of the shape.
/// A limit is a terminal cone: every cone admits a unique cone morphism into it.
///
/// As first concrete instances, binary products are the limits of the discrete
/// two-object diagram, and terminal objects are the limits of the empty diagram.

from category.category import Category, category_identity_src, category_identity_dst,
    category_compose_src, category_compose_dst, category_id_left, category_id_right
from category.functor import Functor, is_functor, functor_src_axiom, functor_dst_axiom,
    functor_identity_axiom, functor_compose_axiom, functor_new_src_cat, functor_new_dst_cat,
    functor_new_obj_map, functor_new_mor_map
from category.discrete_category import discrete_category, discrete_category_src_apply,
    discrete_category_dst_apply, discrete_category_identity_apply,
    discrete_category_compose_apply
from category.terminal_category import terminal_category, UnitType, unit_type_unique
from category.category_iso import Iso, is_iso_pair, iso_new_cat, iso_new_hom, iso_new_inv

// ---------------------------------------------------------------------------
// Cones over diagrams
// ---------------------------------------------------------------------------

/// True if `legs` makes `n` the apex of a cone over the diagram `d` in `c`.
/// Each `legs(i)` is a morphism from `n` to `d.obj_map(i)`, and every triangle
/// `compose(d.mor_map(f), legs(src(f))) = legs(dst(f))` commutes.
define is_cone[O, M, JO, JM](
    c: Category[O, M],
    d: Functor[JO, JM, O, M],
    n: O,
    legs: JO -> M
) -> Bool {
    forall(i: JO) { c.src(legs(i)) = n }
    and forall(i: JO) { c.dst(legs(i)) = d.obj_map(i) }
    and forall(f: JM) {
        c.compose(d.mor_map(f), legs(d.src_cat.src(f))) = legs(d.src_cat.dst(f))
    }
}

/// The apex of a cone is the source of every leg.
theorem cone_apply_src[O, M, JO, JM](
    c: Category[O, M],
    d: Functor[JO, JM, O, M],
    n: O,
    legs: JO -> M,
    i: JO
) {
    is_cone(c, d, n, legs) implies c.src(legs(i)) = n
} by {
    is_cone(c, d, n, legs) =
        (forall(j: JO) { c.src(legs(j)) = n }
         and forall(j: JO) { c.dst(legs(j)) = d.obj_map(j) }
         and forall(f: JM) {
             c.compose(d.mor_map(f), legs(d.src_cat.src(f))) = legs(d.src_cat.dst(f))
         })
    forall(j: JO) {
        c.src(legs(j)) = n
    }
}

/// The target of a leg of a cone is the corresponding object of the diagram.
theorem cone_apply_dst[O, M, JO, JM](
    c: Category[O, M],
    d: Functor[JO, JM, O, M],
    n: O,
    legs: JO -> M,
    i: JO
) {
    is_cone(c, d, n, legs) implies c.dst(legs(i)) = d.obj_map(i)
} by {
    is_cone(c, d, n, legs) =
        (forall(j: JO) { c.src(legs(j)) = n }
         and forall(j: JO) { c.dst(legs(j)) = d.obj_map(j) }
         and forall(f: JM) {
             c.compose(d.mor_map(f), legs(d.src_cat.src(f))) = legs(d.src_cat.dst(f))
         })
    forall(j: JO) {
        c.dst(legs(j)) = d.obj_map(j)
    }
}

/// Every triangle of a cone commutes.
theorem cone_apply_naturality[O, M, JO, JM](
    c: Category[O, M],
    d: Functor[JO, JM, O, M],
    n: O,
    legs: JO -> M,
    f: JM
) {
    is_cone(c, d, n, legs) implies
        c.compose(d.mor_map(f), legs(d.src_cat.src(f))) = legs(d.src_cat.dst(f))
} by {
    is_cone(c, d, n, legs) =
        (forall(j: JO) { c.src(legs(j)) = n }
         and forall(j: JO) { c.dst(legs(j)) = d.obj_map(j) }
         and forall(g: JM) {
             c.compose(d.mor_map(g), legs(d.src_cat.src(g))) = legs(d.src_cat.dst(g))
         })
    forall(g: JM) {
        c.compose(d.mor_map(g), legs(d.src_cat.src(g))) = legs(d.src_cat.dst(g))
    }
}

/// Pointwise source, target and triangle conditions combine into a cone.
theorem cone_intro[O, M, JO, JM](
    c: Category[O, M],
    d: Functor[JO, JM, O, M],
    n: O,
    legs: JO -> M
) {
    (forall(i: JO) { c.src(legs(i)) = n })
    and (forall(i: JO) { c.dst(legs(i)) = d.obj_map(i) })
    and (forall(f: JM) {
        c.compose(d.mor_map(f), legs(d.src_cat.src(f))) = legs(d.src_cat.dst(f))
    })
    implies is_cone(c, d, n, legs)
} by {
    if (forall(i: JO) { c.src(legs(i)) = n })
        and (forall(i: JO) { c.dst(legs(i)) = d.obj_map(i) })
        and (forall(f: JM) {
            c.compose(d.mor_map(f), legs(d.src_cat.src(f))) = legs(d.src_cat.dst(f))
        }) {
        is_cone(c, d, n, legs) =
            (forall(j: JO) { c.src(legs(j)) = n }
             and forall(j: JO) { c.dst(legs(j)) = d.obj_map(j) }
             and forall(g: JM) {
                 c.compose(d.mor_map(g), legs(d.src_cat.src(g))) = legs(d.src_cat.dst(g))
             })
        is_cone(c, d, n, legs)
    }
}

/// True if `f` is a morphism of cones from the cone `(m, mlegs)` to the cone
/// `(n, legs)`: it is a morphism from `m` to `n` and every triangle
/// `compose(legs(i), f) = mlegs(i)` commutes.
define is_cone_map[O, M, JO, JM](
    c: Category[O, M],
    d: Functor[JO, JM, O, M],
    m: O,
    mlegs: JO -> M,
    n: O,
    legs: JO -> M,
    f: M
) -> Bool {
    c.src(f) = m
    and c.dst(f) = n
    and forall(i: JO) { c.compose(legs(i), f) = mlegs(i) }
}

/// The source of a cone morphism is the apex of the source cone.
theorem cone_map_apply_src[O, M, JO, JM](
    c: Category[O, M],
    d: Functor[JO, JM, O, M],
    m: O,
    mlegs: JO -> M,
    n: O,
    legs: JO -> M,
    f: M
) {
    is_cone_map(c, d, m, mlegs, n, legs, f) implies c.src(f) = m
} by {
    is_cone_map(c, d, m, mlegs, n, legs, f) =
        (c.src(f) = m and c.dst(f) = n and forall(i: JO) { c.compose(legs(i), f) = mlegs(i) })
}

/// The target of a cone morphism is the apex of the target cone.
theorem cone_map_apply_dst[O, M, JO, JM](
    c: Category[O, M],
    d: Functor[JO, JM, O, M],
    m: O,
    mlegs: JO -> M,
    n: O,
    legs: JO -> M,
    f: M
) {
    is_cone_map(c, d, m, mlegs, n, legs, f) implies c.dst(f) = n
} by {
    is_cone_map(c, d, m, mlegs, n, legs, f) =
        (c.src(f) = m and c.dst(f) = n and forall(i: JO) { c.compose(legs(i), f) = mlegs(i) })
}

/// Every triangle of a cone morphism commutes.
theorem cone_map_apply_legs[O, M, JO, JM](
    c: Category[O, M],
    d: Functor[JO, JM, O, M],
    m: O,
    mlegs: JO -> M,
    n: O,
    legs: JO -> M,
    f: M,
    i: JO
) {
    is_cone_map(c, d, m, mlegs, n, legs, f) implies c.compose(legs(i), f) = mlegs(i)
} by {
    is_cone_map(c, d, m, mlegs, n, legs, f) =
        (c.src(f) = m and c.dst(f) = n and forall(j: JO) { c.compose(legs(j), f) = mlegs(j) })
    forall(j: JO) {
        c.compose(legs(j), f) = mlegs(j)
    }
}

// ---------------------------------------------------------------------------
// Limits: terminal cones
// ---------------------------------------------------------------------------

/// True if the cone `(n, legs)` over the diagram `d` in `c` is a limit:
/// it is a cone, every cone admits a cone morphism into it, and any two cone
/// morphisms into it are equal.
define is_limit[O, M, JO, JM](
    c: Category[O, M],
    d: Functor[JO, JM, O, M],
    n: O,
    legs: JO -> M
) -> Bool {
    is_cone(c, d, n, legs)
    and forall(m: O, mlegs: JO -> M) {
        is_cone(c, d, m, mlegs) implies exists(f: M) {
            is_cone_map(c, d, m, mlegs, n, legs, f)
        }
    }
    and forall(m: O, mlegs: JO -> M, f: M, g: M) {
        is_cone(c, d, m, mlegs)
        and is_cone_map(c, d, m, mlegs, n, legs, f)
        and is_cone_map(c, d, m, mlegs, n, legs, g)
        implies f = g
    }
}

/// The cone underlying a limit is a cone.
theorem limit_is_cone[O, M, JO, JM](
    c: Category[O, M],
    d: Functor[JO, JM, O, M],
    n: O,
    legs: JO -> M
) {
    is_limit(c, d, n, legs) implies is_cone(c, d, n, legs)
} by {
    is_limit(c, d, n, legs) =
        (is_cone(c, d, n, legs)
         and forall(m: O, mlegs: JO -> M) {
             is_cone(c, d, m, mlegs) implies exists(f: M) {
                 is_cone_map(c, d, m, mlegs, n, legs, f)
             }
         }
         and forall(m: O, mlegs: JO -> M, f: M, g: M) {
             is_cone(c, d, m, mlegs)
             and is_cone_map(c, d, m, mlegs, n, legs, f)
             and is_cone_map(c, d, m, mlegs, n, legs, g)
             implies f = g
         })
}

/// A cone together with existence and uniqueness of cone morphisms into it is a
/// limit.
theorem is_limit_intro[O, M, JO, JM](
    c: Category[O, M],
    d: Functor[JO, JM, O, M],
    n: O,
    legs: JO -> M
) {
    is_cone(c, d, n, legs)
    and (forall(m: O, mlegs: JO -> M) {
        is_cone(c, d, m, mlegs) implies exists(f: M) {
            is_cone_map(c, d, m, mlegs, n, legs, f)
        }
    })
    and (forall(m: O, mlegs: JO -> M, f: M, g: M) {
        is_cone(c, d, m, mlegs)
        and is_cone_map(c, d, m, mlegs, n, legs, f)
        and is_cone_map(c, d, m, mlegs, n, legs, g)
        implies f = g
    })
    implies is_limit(c, d, n, legs)
} by {
    if is_cone(c, d, n, legs)
        and (forall(m: O, mlegs: JO -> M) {
            is_cone(c, d, m, mlegs) implies exists(f: M) {
                is_cone_map(c, d, m, mlegs, n, legs, f)
            }
        })
        and (forall(m: O, mlegs: JO -> M, f: M, g: M) {
            is_cone(c, d, m, mlegs)
            and is_cone_map(c, d, m, mlegs, n, legs, f)
            and is_cone_map(c, d, m, mlegs, n, legs, g)
            implies f = g
        }) {
        is_limit(c, d, n, legs) =
            (is_cone(c, d, n, legs)
             and forall(x: O, xlegs: JO -> M) {
                 is_cone(c, d, x, xlegs) implies exists(f: M) {
                     is_cone_map(c, d, x, xlegs, n, legs, f)
                 }
             }
             and forall(x: O, xlegs: JO -> M, f: M, g: M) {
                 is_cone(c, d, x, xlegs)
                 and is_cone_map(c, d, x, xlegs, n, legs, f)
                 and is_cone_map(c, d, x, xlegs, n, legs, g)
                 implies f = g
             })
        is_limit(c, d, n, legs)
    }
}

/// Every cone over the diagram of a limit admits a cone morphism into the limit.
theorem limit_existence[O, M, JO, JM](
    c: Category[O, M],
    d: Functor[JO, JM, O, M],
    n: O,
    legs: JO -> M,
    m: O,
    mlegs: JO -> M
) {
    is_limit(c, d, n, legs) and is_cone(c, d, m, mlegs) implies
        exists(f: M) { is_cone_map(c, d, m, mlegs, n, legs, f) }
} by {
    if is_limit(c, d, n, legs) and is_cone(c, d, m, mlegs) {
        is_limit(c, d, n, legs) =
            (is_cone(c, d, n, legs)
             and forall(x: O, xlegs: JO -> M) {
                 is_cone(c, d, x, xlegs) implies exists(f: M) {
                     is_cone_map(c, d, x, xlegs, n, legs, f)
                 }
             }
             and forall(x: O, xlegs: JO -> M, f: M, g: M) {
                 is_cone(c, d, x, xlegs)
                 and is_cone_map(c, d, x, xlegs, n, legs, f)
                 and is_cone_map(c, d, x, xlegs, n, legs, g)
                 implies f = g
             })
        is_cone(c, d, m, mlegs) implies exists(f: M) {
            is_cone_map(c, d, m, mlegs, n, legs, f)
        }
        exists(f: M) { is_cone_map(c, d, m, mlegs, n, legs, f) }
    }
}

/// Any two cone morphisms into a limit are equal.
theorem limit_uniqueness[O, M, JO, JM](
    c: Category[O, M],
    d: Functor[JO, JM, O, M],
    n: O,
    legs: JO -> M,
    m: O,
    mlegs: JO -> M,
    f: M,
    g: M
) {
    is_limit(c, d, n, legs)
    and is_cone(c, d, m, mlegs)
    and is_cone_map(c, d, m, mlegs, n, legs, f)
    and is_cone_map(c, d, m, mlegs, n, legs, g)
    implies f = g
} by {
    if is_limit(c, d, n, legs)
        and is_cone(c, d, m, mlegs)
        and is_cone_map(c, d, m, mlegs, n, legs, f)
        and is_cone_map(c, d, m, mlegs, n, legs, g) {
        is_limit(c, d, n, legs) =
            (is_cone(c, d, n, legs)
             and forall(x: O, xlegs: JO -> M) {
                 is_cone(c, d, x, xlegs) implies exists(h: M) {
                     is_cone_map(c, d, x, xlegs, n, legs, h)
                 }
             }
             and forall(x: O, xlegs: JO -> M, h: M, k: M) {
                 is_cone(c, d, x, xlegs)
                 and is_cone_map(c, d, x, xlegs, n, legs, h)
                 and is_cone_map(c, d, x, xlegs, n, legs, k)
                 implies h = k
             })
        forall(h: M, k: M) {
            (is_cone(c, d, m, mlegs)
             and is_cone_map(c, d, m, mlegs, n, legs, h)
             and is_cone_map(c, d, m, mlegs, n, legs, k)
             implies h = k)
        }
        f = g
    }
}

// ---------------------------------------------------------------------------
// Terminal and initial objects
// ---------------------------------------------------------------------------

/// True if `t` is a terminal object of `c`: every object admits a unique
/// morphism into `t`. A terminal object is the limit of the empty diagram.
define is_terminal[O, M](c: Category[O, M], t: O) -> Bool {
    forall(x: O) {
        exists(f: M) {
            c.src(f) = x
            and c.dst(f) = t
            and forall(g: M) {
                c.src(g) = x and c.dst(g) = t implies g = f
            }
        }
    }
}

/// Every object admits a morphism into a terminal object.
theorem terminal_arrow_exists[O, M](c: Category[O, M], t: O, x: O) {
    is_terminal(c, t) implies exists(f: M) { c.src(f) = x and c.dst(f) = t }
} by {
    if is_terminal(c, t) {
        is_terminal(c, t) = forall(y: O) {
            exists(f: M) {
                c.src(f) = y
                and c.dst(f) = t
                and forall(g: M) {
                    c.src(g) = y and c.dst(g) = t implies g = f
                }
            }
        }
        exists(f: M) {
            c.src(f) = x
            and c.dst(f) = t
            and forall(g: M) {
                c.src(g) = x and c.dst(g) = t implies g = f
            }
        }
        exists(f: M) { c.src(f) = x and c.dst(f) = t }
    }
}

/// Any two morphisms into a terminal object are equal.
theorem terminal_arrow_unique[O, M](c: Category[O, M], t: O, f: M, g: M, x: O) {
    is_terminal(c, t)
    and c.src(f) = x and c.dst(f) = t
    and c.src(g) = x and c.dst(g) = t
    implies f = g
} by {
    if is_terminal(c, t)
        and c.src(f) = x and c.dst(f) = t
        and c.src(g) = x and c.dst(g) = t {
        is_terminal(c, t) = forall(y: O) {
            exists(h: M) {
                c.src(h) = y
                and c.dst(h) = t
                and forall(k: M) {
                    c.src(k) = y and c.dst(k) = t implies k = h
                }
            }
        }
        exists(h: M) {
            c.src(h) = x
            and c.dst(h) = t
            and forall(k: M) {
                c.src(k) = x and c.dst(k) = t implies k = h
            }
        }
        let h: M satisfy {
            c.src(h) = x
            and c.dst(h) = t
            and forall(k: M) {
                c.src(k) = x and c.dst(k) = t implies k = h
            }
        }
        forall(k: M) {
            c.src(k) = x and c.dst(k) = t implies k = h
        }
        f = h
        g = h
        f = g
    }
}

/// True if `i` is an initial object of `c`: every object admits a unique
/// morphism out of `i`. An initial object is the colimit of the empty diagram.
define is_initial[O, M](c: Category[O, M], i: O) -> Bool {
    forall(x: O) {
        exists(f: M) {
            c.src(f) = i
            and c.dst(f) = x
            and forall(g: M) {
                c.src(g) = i and c.dst(g) = x implies g = f
            }
        }
    }
}

/// Every object admits a morphism out of an initial object.
theorem initial_arrow_exists[O, M](c: Category[O, M], i: O, x: O) {
    is_initial(c, i) implies exists(f: M) { c.src(f) = i and c.dst(f) = x }
} by {
    if is_initial(c, i) {
        is_initial(c, i) = forall(y: O) {
            exists(f: M) {
                c.src(f) = i
                and c.dst(f) = y
                and forall(g: M) {
                    c.src(g) = i and c.dst(g) = y implies g = f
                }
            }
        }
        exists(f: M) {
            c.src(f) = i
            and c.dst(f) = x
            and forall(g: M) {
                c.src(g) = i and c.dst(g) = x implies g = f
            }
        }
        exists(f: M) { c.src(f) = i and c.dst(f) = x }
    }
}

/// Any two morphisms out of an initial object are equal.
theorem initial_arrow_unique[O, M](c: Category[O, M], i: O, f: M, g: M, x: O) {
    is_initial(c, i)
    and c.src(f) = i and c.dst(f) = x
    and c.src(g) = i and c.dst(g) = x
    implies f = g
} by {
    if is_initial(c, i)
        and c.src(f) = i and c.dst(f) = x
        and c.src(g) = i and c.dst(g) = x {
        is_initial(c, i) = forall(y: O) {
            exists(h: M) {
                c.src(h) = i
                and c.dst(h) = y
                and forall(k: M) {
                    c.src(k) = i and c.dst(k) = y implies k = h
                }
            }
        }
        exists(h: M) {
            c.src(h) = i
            and c.dst(h) = x
            and forall(k: M) {
                c.src(k) = i and c.dst(k) = x implies k = h
            }
        }
        let h: M satisfy {
            c.src(h) = i
            and c.dst(h) = x
            and forall(k: M) {
                c.src(k) = i and c.dst(k) = x implies k = h
            }
        }
        forall(k: M) {
            c.src(k) = i and c.dst(k) = x implies k = h
        }
        f = h
        g = h
        f = g
    }
}

/// The unique object of the terminal category is terminal.
theorem terminal_category_terminal {
    is_terminal(terminal_category, UnitType.point)
} by {
    is_terminal(terminal_category, UnitType.point) = forall(x: UnitType) {
        exists(f: UnitType) {
            terminal_category.src(f) = x
            and terminal_category.dst(f) = UnitType.point
            and forall(g: UnitType) {
                terminal_category.src(g) = x and terminal_category.dst(g) = UnitType.point
                    implies g = f
            }
        }
    }
    forall(x: UnitType) {
        unit_type_unique(x)
        x = UnitType.point
        terminal_category.src(UnitType.point) = UnitType.point
        terminal_category.dst(UnitType.point) = UnitType.point
        exists(f: UnitType) {
            terminal_category.src(f) = x
            and terminal_category.dst(f) = UnitType.point
            and forall(g: UnitType) {
                terminal_category.src(g) = x and terminal_category.dst(g) = UnitType.point
                    implies g = f
            }
        }
    }
}

/// The unique object of the terminal category is initial.
theorem terminal_category_initial {
    is_initial(terminal_category, UnitType.point)
} by {
    is_initial(terminal_category, UnitType.point) = forall(x: UnitType) {
        exists(f: UnitType) {
            terminal_category.src(f) = UnitType.point
            and terminal_category.dst(f) = x
            and forall(g: UnitType) {
                terminal_category.src(g) = UnitType.point and terminal_category.dst(g) = x
                    implies g = f
            }
        }
    }
    forall(x: UnitType) {
        unit_type_unique(x)
        x = UnitType.point
        terminal_category.src(UnitType.point) = UnitType.point
        terminal_category.dst(UnitType.point) = UnitType.point
        exists(f: UnitType) {
            terminal_category.src(f) = UnitType.point
            and terminal_category.dst(f) = x
            and forall(g: UnitType) {
                terminal_category.src(g) = UnitType.point and terminal_category.dst(g) = x
                    implies g = f
            }
        }
    }
}

/// Any two terminal objects are isomorphic: the unique arrows between them are
/// mutually inverse.
theorem terminal_objects_iso[O, M](c: Category[O, M], t: O, s: O) {
    is_terminal(c, t) and is_terminal(c, s) implies
        exists(e: Iso[O, M]) {
            e.cat = c and e.cat.src(e.hom) = s and e.cat.dst(e.hom) = t
        }
} by {
    if is_terminal(c, t) and is_terminal(c, s) {
        terminal_arrow_exists(c, t, s)
        let f: M satisfy { c.src(f) = s and c.dst(f) = t }
        terminal_arrow_exists(c, s, t)
        let g: M satisfy { c.src(g) = t and c.dst(g) = s }

        c.src(f) = c.dst(g)
        category_compose_src(c, f, g)
        c.src(c.compose(f, g)) = c.src(g)
        c.src(c.compose(f, g)) = t
        c.src(g) = c.dst(f)
        category_compose_dst(c, f, g)
        c.dst(c.compose(f, g)) = c.dst(f)
        c.dst(c.compose(f, g)) = t
        terminal_arrow_unique(c, t, c.compose(f, g), c.identity(t), t)
        c.compose(f, g) = c.identity(t)

        c.src(g) = c.dst(f)
        category_compose_src(c, g, f)
        c.src(c.compose(g, f)) = c.src(f)
        c.src(c.compose(g, f)) = s
        c.src(f) = c.dst(g)
        category_compose_dst(c, g, f)
        c.dst(c.compose(g, f)) = c.dst(g)
        c.dst(c.compose(g, f)) = s
        terminal_arrow_unique(c, s, c.compose(g, f), c.identity(s), s)
        c.compose(g, f) = c.identity(s)

        c.src(f) = c.dst(g)
        c.src(g) = c.dst(f)
        c.compose(g, f) = c.identity(c.src(f))
        c.compose(f, g) = c.identity(c.dst(f))
        is_iso_pair(c, f, g)

        let e: Iso[O, M] satisfy {
            Iso[O, M].new(c, f, g) = Option.some(e)
        }
        iso_new_cat(c, f, g, e)
        iso_new_hom(c, f, g, e)
        e.cat = c
        e.hom = f
        e.cat.src(e.hom) = s
        e.cat.dst(e.hom) = t
        exists(e0: Iso[O, M]) {
            e0.cat = c and e0.cat.src(e0.hom) = s and e0.cat.dst(e0.hom) = t
        }
    }
}

// ---------------------------------------------------------------------------
// Binary products
// ---------------------------------------------------------------------------

/// True if `h` is the mediating morphism of the product: it goes from `x` to
/// `p` and the two triangles with the projections commute.
define product_triangle[O, M](
    c: Category[O, M],
    a: O,
    b: O,
    p: O,
    proj1: M,
    proj2: M,
    x: O,
    f: M,
    g: M,
    h: M
) -> Bool {
    c.src(h) = x and c.dst(h) = p
    and c.compose(proj1, h) = f
    and c.compose(proj2, h) = g
}

/// True if `(p, proj1, proj2)` is a product of `a` and `b` in `c`: `proj1` and
/// `proj2` are projections from `p` to `a` and `b`, and every pair of morphisms
/// `f: x -> a`, `g: x -> b` factors uniquely through `p`.
define is_product[O, M](
    c: Category[O, M],
    a: O,
    b: O,
    p: O,
    proj1: M,
    proj2: M
) -> Bool {
    c.src(proj1) = p and c.dst(proj1) = a
    and c.src(proj2) = p and c.dst(proj2) = b
    and forall(x: O, f: M, g: M) {
        c.src(f) = x and c.dst(f) = a and c.src(g) = x and c.dst(g) = b implies
        exists(h: M) {
            product_triangle(c, a, b, p, proj1, proj2, x, f, g, h)
            and forall(k: M) {
                product_triangle(c, a, b, p, proj1, proj2, x, f, g, k) implies k = h
            }
        }
    }
}

/// The endpoint conditions and the unique factoring property combine into a
/// product.
theorem is_product_intro[O, M](
    c: Category[O, M],
    a: O,
    b: O,
    p: O,
    proj1: M,
    proj2: M
) {
    c.src(proj1) = p and c.dst(proj1) = a
    and c.src(proj2) = p and c.dst(proj2) = b
    and (forall(x: O, f: M, g: M) {
        c.src(f) = x and c.dst(f) = a and c.src(g) = x and c.dst(g) = b implies
        exists(h: M) {
            product_triangle(c, a, b, p, proj1, proj2, x, f, g, h)
            and forall(k: M) {
                product_triangle(c, a, b, p, proj1, proj2, x, f, g, k) implies k = h
            }
        }
    })
    implies is_product(c, a, b, p, proj1, proj2)
} by {
    if c.src(proj1) = p and c.dst(proj1) = a
        and c.src(proj2) = p and c.dst(proj2) = b
        and (forall(x: O, f: M, g: M) {
            c.src(f) = x and c.dst(f) = a and c.src(g) = x and c.dst(g) = b implies
            exists(h: M) {
                product_triangle(c, a, b, p, proj1, proj2, x, f, g, h)
                and forall(k: M) {
                    product_triangle(c, a, b, p, proj1, proj2, x, f, g, k) implies k = h
                }
            }
        }) {
        is_product(c, a, b, p, proj1, proj2) =
            (c.src(proj1) = p and c.dst(proj1) = a
             and c.src(proj2) = p and c.dst(proj2) = b
             and forall(x0: O, f0: M, g0: M) {
                 c.src(f0) = x0 and c.dst(f0) = a and c.src(g0) = x0 and c.dst(g0) = b implies
                 exists(h: M) {
                     product_triangle(c, a, b, p, proj1, proj2, x0, f0, g0, h)
                     and forall(k: M) {
                         product_triangle(c, a, b, p, proj1, proj2, x0, f0, g0, k) implies k = h
                     }
                 }
             })
        is_product(c, a, b, p, proj1, proj2)
    }
}

/// The first projection of a product has the product as its source.
theorem product_proj1_src[O, M](c: Category[O, M], a: O, b: O, p: O, proj1: M, proj2: M) {
    is_product(c, a, b, p, proj1, proj2) implies c.src(proj1) = p
} by {
    is_product(c, a, b, p, proj1, proj2) =
        (c.src(proj1) = p and c.dst(proj1) = a
         and c.src(proj2) = p and c.dst(proj2) = b
         and forall(x: O, f: M, g: M) {
             c.src(f) = x and c.dst(f) = a and c.src(g) = x and c.dst(g) = b implies
             exists(h: M) {
                 product_triangle(c, a, b, p, proj1, proj2, x, f, g, h)
                 and forall(k: M) {
                     product_triangle(c, a, b, p, proj1, proj2, x, f, g, k) implies k = h
                 }
             }
         })
}

/// The first projection of a product targets the first factor.
theorem product_proj1_dst[O, M](c: Category[O, M], a: O, b: O, p: O, proj1: M, proj2: M) {
    is_product(c, a, b, p, proj1, proj2) implies c.dst(proj1) = a
} by {
    is_product(c, a, b, p, proj1, proj2) =
        (c.src(proj1) = p and c.dst(proj1) = a
         and c.src(proj2) = p and c.dst(proj2) = b
         and forall(x: O, f: M, g: M) {
             c.src(f) = x and c.dst(f) = a and c.src(g) = x and c.dst(g) = b implies
             exists(h: M) {
                 product_triangle(c, a, b, p, proj1, proj2, x, f, g, h)
                 and forall(k: M) {
                     product_triangle(c, a, b, p, proj1, proj2, x, f, g, k) implies k = h
                 }
             }
         })
}

/// The second projection of a product has the product as its source.
theorem product_proj2_src[O, M](c: Category[O, M], a: O, b: O, p: O, proj1: M, proj2: M) {
    is_product(c, a, b, p, proj1, proj2) implies c.src(proj2) = p
} by {
    is_product(c, a, b, p, proj1, proj2) =
        (c.src(proj1) = p and c.dst(proj1) = a
         and c.src(proj2) = p and c.dst(proj2) = b
         and forall(x: O, f: M, g: M) {
             c.src(f) = x and c.dst(f) = a and c.src(g) = x and c.dst(g) = b implies
             exists(h: M) {
                 product_triangle(c, a, b, p, proj1, proj2, x, f, g, h)
                 and forall(k: M) {
                     product_triangle(c, a, b, p, proj1, proj2, x, f, g, k) implies k = h
                 }
             }
         })
}

/// The second projection of a product targets the second factor.
theorem product_proj2_dst[O, M](c: Category[O, M], a: O, b: O, p: O, proj1: M, proj2: M) {
    is_product(c, a, b, p, proj1, proj2) implies c.dst(proj2) = b
} by {
    is_product(c, a, b, p, proj1, proj2) =
        (c.src(proj1) = p and c.dst(proj1) = a
         and c.src(proj2) = p and c.dst(proj2) = b
         and forall(x: O, f: M, g: M) {
             c.src(f) = x and c.dst(f) = a and c.src(g) = x and c.dst(g) = b implies
             exists(h: M) {
                 product_triangle(c, a, b, p, proj1, proj2, x, f, g, h)
                 and forall(k: M) {
                     product_triangle(c, a, b, p, proj1, proj2, x, f, g, k) implies k = h
                 }
             }
         })
}

/// A pair of morphisms into the factors of a product factors through the product.
theorem product_map_exists[O, M](
    c: Category[O, M], a: O, b: O, p: O, proj1: M, proj2: M, x: O, f: M, g: M
) {
    is_product(c, a, b, p, proj1, proj2)
    and c.src(f) = x and c.dst(f) = a
    and c.src(g) = x and c.dst(g) = b
    implies exists(h: M) { product_triangle(c, a, b, p, proj1, proj2, x, f, g, h) }
} by {
    if is_product(c, a, b, p, proj1, proj2)
        and c.src(f) = x and c.dst(f) = a
        and c.src(g) = x and c.dst(g) = b {
        is_product(c, a, b, p, proj1, proj2) =
            (c.src(proj1) = p and c.dst(proj1) = a
             and c.src(proj2) = p and c.dst(proj2) = b
             and forall(x0: O, f0: M, g0: M) {
                 c.src(f0) = x0 and c.dst(f0) = a and c.src(g0) = x0 and c.dst(g0) = b implies
                 exists(h: M) {
                     product_triangle(c, a, b, p, proj1, proj2, x0, f0, g0, h)
                     and forall(k: M) {
                         product_triangle(c, a, b, p, proj1, proj2, x0, f0, g0, k) implies k = h
                     }
                 }
             })
        exists(h: M) {
            product_triangle(c, a, b, p, proj1, proj2, x, f, g, h)
            and forall(k: M) {
                product_triangle(c, a, b, p, proj1, proj2, x, f, g, k) implies k = h
            }
        }
        exists(h: M) { product_triangle(c, a, b, p, proj1, proj2, x, f, g, h) }
    }
}

/// The mediating morphism of a product is unique.
theorem product_map_unique[O, M](
    c: Category[O, M], a: O, b: O, p: O, proj1: M, proj2: M,
    x: O, f: M, g: M, h: M, k: M
) {
    is_product(c, a, b, p, proj1, proj2)
    and product_triangle(c, a, b, p, proj1, proj2, x, f, g, h)
    and product_triangle(c, a, b, p, proj1, proj2, x, f, g, k)
    implies h = k
} by {
    if is_product(c, a, b, p, proj1, proj2)
        and product_triangle(c, a, b, p, proj1, proj2, x, f, g, h)
        and product_triangle(c, a, b, p, proj1, proj2, x, f, g, k) {
        product_triangle(c, a, b, p, proj1, proj2, x, f, g, h) =
            (c.src(h) = x and c.dst(h) = p
             and c.compose(proj1, h) = f
             and c.compose(proj2, h) = g)
        c.src(h) = x
        c.dst(h) = p
        c.compose(proj1, h) = f
        c.compose(proj2, h) = g

        is_product(c, a, b, p, proj1, proj2) =
            (c.src(proj1) = p and c.dst(proj1) = a
             and c.src(proj2) = p and c.dst(proj2) = b
             and forall(x0: O, f0: M, g0: M) {
                 c.src(f0) = x0 and c.dst(f0) = a and c.src(g0) = x0 and c.dst(g0) = b implies
                 exists(h0: M) {
                     product_triangle(c, a, b, p, proj1, proj2, x0, f0, g0, h0)
                     and forall(k0: M) {
                         product_triangle(c, a, b, p, proj1, proj2, x0, f0, g0, k0)
                             implies k0 = h0
                     }
                 }
             })
        c.src(proj1) = p
        c.dst(proj1) = a
        c.src(proj2) = p
        c.dst(proj2) = b

        // The triangles force the endpoints of f and g.
        c.src(proj1) = c.dst(h)
        category_compose_src(c, proj1, h)
        c.src(c.compose(proj1, h)) = c.src(h)
        c.src(f) = x
        category_compose_dst(c, proj1, h)
        c.dst(c.compose(proj1, h)) = c.dst(proj1)
        c.dst(f) = a
        c.src(proj2) = c.dst(h)
        category_compose_src(c, proj2, h)
        c.src(c.compose(proj2, h)) = c.src(h)
        c.src(g) = x
        category_compose_dst(c, proj2, h)
        c.dst(c.compose(proj2, h)) = c.dst(proj2)
        c.dst(g) = b

        exists(h0: M) {
            product_triangle(c, a, b, p, proj1, proj2, x, f, g, h0)
            and forall(k0: M) {
                product_triangle(c, a, b, p, proj1, proj2, x, f, g, k0) implies k0 = h0
            }
        }
        let h0: M satisfy {
            product_triangle(c, a, b, p, proj1, proj2, x, f, g, h0)
            and forall(k0: M) {
                product_triangle(c, a, b, p, proj1, proj2, x, f, g, k0) implies k0 = h0
            }
        }
        forall(k0: M) {
            product_triangle(c, a, b, p, proj1, proj2, x, f, g, k0) implies k0 = h0
        }
        h = h0
        k = h0
        h = k
    }
}

// ---------------------------------------------------------------------------
// Products are limits of the discrete two-object diagram
// ---------------------------------------------------------------------------
//
// The discrete category over `Bool` has the two objects `true` and `false` and
// only identity morphisms. A diagram of shape `discrete_category[Bool](true)`
// is determined by the two objects `a = obj_map(true)` and `b = obj_map(false)`,
// and a limit of this diagram is exactly a product of `a` and `b`.

/// The object map of the two-object diagram: `true` maps to `a`, `false` maps to `b`.
define pair_obj[O](a: O, b: O, x: Bool) -> O {
    if x { a } else { b }
}

/// The morphism map of the two-object diagram: every morphism of the discrete
/// shape is sent to the identity on the image object.
define pair_mor[O, M](c: Category[O, M], a: O, b: O, f: Bool) -> M {
    c.identity(pair_obj(a, b, f))
}

/// The object map of the two-object diagram sends `true` to `a`.
theorem pair_obj_true[O](a: O, b: O) {
    pair_obj(a, b, true) = a
}

/// The object map of the two-object diagram sends `false` to `b`.
theorem pair_obj_false[O](a: O, b: O) {
    pair_obj(a, b, false) = b
}

/// The pair maps form a functor from the discrete category on `Bool` into `c`.
theorem pair_is_functor[O, M](c: Category[O, M], a: O, b: O) {
    is_functor(discrete_category[Bool](true), c,
        function(x: Bool) { pair_obj(a, b, x) },
        function(f: Bool) { pair_mor(c, a, b, f) })
} by {
    let om: Bool -> O = function(x: Bool) { pair_obj(a, b, x) }
    let mm: Bool -> M = function(f: Bool) { pair_mor(c, a, b, f) }
    forall(x: Bool) {
        om(x) = pair_obj(a, b, x)
    }
    forall(f: Bool) {
        mm(f) = pair_mor(c, a, b, f)
        mm(f) = c.identity(pair_obj(a, b, f))
    }
    forall(f: Bool) {
        category_identity_src(c, pair_obj(a, b, f))
        c.src(mm(f)) = pair_obj(a, b, f)
        discrete_category_src_apply(true, f)
        discrete_category(true).src(f) = f
        pair_obj(a, b, f) = om(discrete_category(true).src(f))
        c.src(mm(f)) = om(discrete_category(true).src(f))
    }
    functor_src_axiom(discrete_category[Bool](true), c, om, mm)
    forall(f: Bool) {
        category_identity_dst(c, pair_obj(a, b, f))
        c.dst(mm(f)) = pair_obj(a, b, f)
        discrete_category_dst_apply(true, f)
        discrete_category(true).dst(f) = f
        pair_obj(a, b, f) = om(discrete_category(true).dst(f))
        c.dst(mm(f)) = om(discrete_category(true).dst(f))
    }
    functor_dst_axiom(discrete_category[Bool](true), c, om, mm)
    forall(x: Bool) {
        discrete_category_identity_apply(true, x)
        discrete_category(true).identity(x) = x
        mm(discrete_category(true).identity(x)) = mm(x)
        mm(x) = c.identity(pair_obj(a, b, x))
        om(x) = pair_obj(a, b, x)
        mm(discrete_category(true).identity(x)) = c.identity(om(x))
    }
    functor_identity_axiom(discrete_category[Bool](true), c, om, mm)
    forall(p: Bool, q: Bool) {
        if discrete_category(true).src(p) = discrete_category(true).dst(q) {
            discrete_category_src_apply(true, p)
            discrete_category_dst_apply(true, q)
            discrete_category(true).src(p) = p
            discrete_category(true).dst(q) = q
            p = q
            discrete_category_compose_apply(true, p, q)
            discrete_category(true).compose(p, q) = p
            mm(discrete_category(true).compose(p, q)) = mm(p)
            mm(p) = c.identity(pair_obj(a, b, p))
            mm(q) = c.identity(pair_obj(a, b, q))
            pair_obj(a, b, q) = pair_obj(a, b, p)
            mm(q) = c.identity(pair_obj(a, b, p))
            category_identity_src(c, pair_obj(a, b, p))
            category_identity_dst(c, pair_obj(a, b, p))
            c.src(mm(p)) = pair_obj(a, b, p)
            c.dst(mm(p)) = pair_obj(a, b, p)
            category_id_left(c, mm(p))
            c.compose(mm(p), mm(q)) = mm(p)
            mm(discrete_category(true).compose(p, q)) = c.compose(mm(p), mm(q))
        }
    }
    functor_compose_axiom(discrete_category[Bool](true), c, om, mm)
    is_functor(discrete_category[Bool](true), c, om, mm)
}

/// The diagram of the pair `(a, b)`: the functor from the discrete category on
/// `Bool` sending `true` to `a` and `false` to `b`.
let pair_diagram[O, M](c: Category[O, M], a: O, b: O) -> result: Functor[Bool, Bool, O, M] satisfy {
    Functor[Bool, Bool, O, M].new(discrete_category[Bool](true), c,
        function(x: Bool) { pair_obj(a, b, x) },
        function(f: Bool) { pair_mor(c, a, b, f) }) = Option.some(result)
} by {
    pair_is_functor(c, a, b)
}

/// The source category of the pair diagram is the discrete category on `Bool`.
theorem pair_diagram_src_cat[O, M](c: Category[O, M], a: O, b: O) {
    pair_diagram(c, a, b).src_cat = discrete_category[Bool](true)
} by {
    let f = pair_diagram(c, a, b)
    (Functor[Bool, Bool, O, M].new(discrete_category[Bool](true), c,
        function(x: Bool) { pair_obj(a, b, x) },
        function(g: Bool) { pair_mor(c, a, b, g) }) = Option.some(f))
    functor_new_src_cat(discrete_category[Bool](true), c,
        function(x: Bool) { pair_obj(a, b, x) },
        function(g: Bool) { pair_mor(c, a, b, g) }, f)
}

/// The object map of the pair diagram sends `true` to `a` and `false` to `b`.
theorem pair_diagram_obj_map[O, M](c: Category[O, M], a: O, b: O) {
    pair_diagram(c, a, b).obj_map = function(x: Bool) { pair_obj(a, b, x) }
} by {
    let f = pair_diagram(c, a, b)
    (Functor[Bool, Bool, O, M].new(discrete_category[Bool](true), c,
        function(x: Bool) { pair_obj(a, b, x) },
        function(g: Bool) { pair_mor(c, a, b, g) }) = Option.some(f))
    functor_new_obj_map(discrete_category[Bool](true), c,
        function(x: Bool) { pair_obj(a, b, x) },
        function(g: Bool) { pair_mor(c, a, b, g) }, f)
}

/// The morphism map of the pair diagram sends every shape morphism to the
/// identity on the image object.
theorem pair_diagram_mor_map[O, M](c: Category[O, M], a: O, b: O) {
    pair_diagram(c, a, b).mor_map = function(f: Bool) { pair_mor(c, a, b, f) }
} by {
    let f = pair_diagram(c, a, b)
    (Functor[Bool, Bool, O, M].new(discrete_category[Bool](true), c,
        function(x: Bool) { pair_obj(a, b, x) },
        function(g: Bool) { pair_mor(c, a, b, g) }) = Option.some(f))
    functor_new_mor_map(discrete_category[Bool](true), c,
        function(x: Bool) { pair_obj(a, b, x) },
        function(g: Bool) { pair_mor(c, a, b, g) }, f)
}

/// The object map of the pair diagram at a shape object.
theorem pair_diagram_obj_apply[O, M](c: Category[O, M], a: O, b: O, x: Bool) {
    pair_diagram(c, a, b).obj_map(x) = pair_obj(a, b, x)
} by {
    pair_diagram_obj_map(c, a, b)
}

/// The pair diagram sends `true` to `a`.
theorem pair_diagram_obj_true[O, M](c: Category[O, M], a: O, b: O) {
    pair_diagram(c, a, b).obj_map(true) = a
} by {
    pair_diagram_obj_apply(c, a, b, true)
    pair_obj_true(a, b)
}

/// The pair diagram sends `false` to `b`.
theorem pair_diagram_obj_false[O, M](c: Category[O, M], a: O, b: O) {
    pair_diagram(c, a, b).obj_map(false) = b
} by {
    pair_diagram_obj_apply(c, a, b, false)
    pair_obj_false(a, b)
}

/// The morphism map of the pair diagram at a shape morphism.
theorem pair_diagram_mor_apply[O, M](c: Category[O, M], a: O, b: O, f: Bool) {
    pair_diagram(c, a, b).mor_map(f) = pair_mor(c, a, b, f)
} by {
    pair_diagram_mor_map(c, a, b)
}

/// The legs of the pair cone: `true` selects the first morphism, `false` the second.
define pair_legs[M](proj1: M, proj2: M, x: Bool) -> M {
    if x { proj1 } else { proj2 }
}

/// The pair legs at `true` select the first morphism.
theorem pair_legs_true[M](proj1: M, proj2: M) {
    pair_legs(proj1, proj2, true) = proj1
}

/// The pair legs at `false` select the second morphism.
theorem pair_legs_false[M](proj1: M, proj2: M) {
    pair_legs(proj1, proj2, false) = proj2
}
