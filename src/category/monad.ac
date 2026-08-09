/// Monads on categories.
///
/// A monad on a category `c` is an endofunctor `t` together with natural
/// transformations `eta : id -> t` (the unit) and `mu : t∘t -> t` (the
/// multiplication) satisfying the associativity law
///   mu ∘ t(mu) = mu ∘ mu_t
/// and the two unit laws
///   mu ∘ t(eta) = id_t   and   mu ∘ eta_t = id_t.
/// Pointwise at an object `x` these read
///   mu_x ∘ t(mu_x) = mu_x ∘ mu_{t(x)},
///   mu_x ∘ t(eta_x) = id_{t(x)},   mu_x ∘ eta_{t(x)} = id_{t(x)}.
///
/// As in the adjunction file, the composite functor `t2 = t ∘ t` is carried
/// explicitly so that the source of `mu` can be named. The laws are stated
/// pointwise, because stating them as equalities of natural transformations
/// would require bundling the whiskered transformations `t(mu)`, `mu_t`,
/// `t(eta)` and `eta_t` with explicit composite functor witnesses through the
/// horizontal-composition API; the pointwise form is equivalent and matches
/// the style of the triangle identities in the adjunction file.

from category.category import Category, category_id_left, category_id_right,
    category_compose_assoc, category_compose_src, category_compose_dst
from category.functor import Functor, identity_functor, identity_functor_src_cat,
    identity_functor_dst_cat, identity_functor_obj_map, identity_functor_mor_map,
    compose_functor, compose_functor_obj_map, compose_functor_mor_map,
    compose_functor_src_cat, compose_functor_dst_cat,
    functor_src, functor_dst, functor_compose
from category.natural_transformation import NaturalTransformation, nat_trans_naturality,
    nat_trans_component_src, nat_trans_component_dst,
    nat_trans_shared_src_cat, nat_trans_shared_dst_cat
from data.basic.functions import compose

/// True if the associativity law holds at the object `x`:
/// `mu_x ∘ t(mu_x) = mu_x ∘ mu_{t(x)}` in the target category.
define monad_associativity_at[O, M](
    t: Functor[O, M, O, M],
    mu: NaturalTransformation[O, M, O, M],
    x: O
) -> Bool {
    t.dst_cat.compose(mu.component(x), t.mor_map(mu.component(x)))
        = t.dst_cat.compose(mu.component(x), mu.component(t.obj_map(x)))
}

/// True if the left unit law holds at the object `x`:
/// `mu_x ∘ t(eta_x) = id_{t(x)}`.
define monad_unit_left_at[O, M](
    t: Functor[O, M, O, M],
    eta: NaturalTransformation[O, M, O, M],
    mu: NaturalTransformation[O, M, O, M],
    x: O
) -> Bool {
    t.dst_cat.compose(mu.component(x), t.mor_map(eta.component(x)))
        = t.dst_cat.identity(t.obj_map(x))
}

/// True if the right unit law holds at the object `x`:
/// `mu_x ∘ eta_{t(x)} = id_{t(x)}`.
define monad_unit_right_at[O, M](
    t: Functor[O, M, O, M],
    eta: NaturalTransformation[O, M, O, M],
    mu: NaturalTransformation[O, M, O, M],
    x: O
) -> Bool {
    t.dst_cat.compose(mu.component(x), eta.component(t.obj_map(x)))
        = t.dst_cat.identity(t.obj_map(x))
}

/// True if `t`, `eta`, `mu` form a monad on `c`, with `t2` the composite `t ∘ t`.
define is_monad[O, M](
    c: Category[O, M],
    t: Functor[O, M, O, M],
    eta: NaturalTransformation[O, M, O, M],
    mu: NaturalTransformation[O, M, O, M],
    t2: Functor[O, M, O, M]
) -> Bool {
    t.src_cat = c and t.dst_cat = c
    and compose_functor(t, t) = Option.some(t2)
    and eta.src_functor = identity_functor(c)
    and eta.dst_functor = t
    and mu.src_functor = t2
    and mu.dst_functor = t
    and forall(x: O) { monad_associativity_at(t, mu, x) }
    and forall(x: O) { monad_unit_left_at(t, eta, mu, x) }
    and forall(x: O) { monad_unit_right_at(t, eta, mu, x) }
}

/// A monad on a category: an endofunctor together with a unit and a
/// multiplication satisfying the monad laws.
structure Monad[O, M] {
    /// The underlying category of the monad.
    category: Category[O, M]

    /// The endofunctor part of the monad.
    functor: Functor[O, M, O, M]

    /// The unit of the monad, a natural transformation from the identity
    /// functor to the endofunctor.
    unit: NaturalTransformation[O, M, O, M]

    /// The multiplication of the monad, a natural transformation from the
    /// composite endofunctor to the endofunctor.
    multiplication: NaturalTransformation[O, M, O, M]

    /// The composite `functor ∘ functor` of the endofunctor with itself.
    composite: Functor[O, M, O, M]
} constraint {
    is_monad(category, functor, unit, multiplication, composite)
}

/// The endofunctor of a monad has the underlying category as its source.
theorem monad_functor_src_cat[O, M](m: Monad[O, M]) {
    m.functor.src_cat = m.category
} by {
    is_monad(m.category, m.functor, m.unit, m.multiplication, m.composite) =
        (m.functor.src_cat = m.category and m.functor.dst_cat = m.category
        and compose_functor(m.functor, m.functor) = Option.some(m.composite)
        and m.unit.src_functor = identity_functor(m.category)
        and m.unit.dst_functor = m.functor
        and m.multiplication.src_functor = m.composite
        and m.multiplication.dst_functor = m.functor
        and forall(x: O) { monad_associativity_at(m.functor, m.multiplication, x) }
        and forall(x: O) { monad_unit_left_at(m.functor, m.unit, m.multiplication, x) }
        and forall(x: O) { monad_unit_right_at(m.functor, m.unit, m.multiplication, x) })
}

/// The endofunctor of a monad has the underlying category as its target.
theorem monad_functor_dst_cat[O, M](m: Monad[O, M]) {
    m.functor.dst_cat = m.category
} by {
    is_monad(m.category, m.functor, m.unit, m.multiplication, m.composite) =
        (m.functor.src_cat = m.category and m.functor.dst_cat = m.category
        and compose_functor(m.functor, m.functor) = Option.some(m.composite)
        and m.unit.src_functor = identity_functor(m.category)
        and m.unit.dst_functor = m.functor
        and m.multiplication.src_functor = m.composite
        and m.multiplication.dst_functor = m.functor
        and forall(x: O) { monad_associativity_at(m.functor, m.multiplication, x) }
        and forall(x: O) { monad_unit_left_at(m.functor, m.unit, m.multiplication, x) }
        and forall(x: O) { monad_unit_right_at(m.functor, m.unit, m.multiplication, x) })
}

/// The composite of the endofunctor of a monad with itself is its composite functor.
theorem monad_compose[O, M](m: Monad[O, M]) {
    compose_functor(m.functor, m.functor) = Option.some(m.composite)
} by {
    is_monad(m.category, m.functor, m.unit, m.multiplication, m.composite) =
        (m.functor.src_cat = m.category and m.functor.dst_cat = m.category
        and compose_functor(m.functor, m.functor) = Option.some(m.composite)
        and m.unit.src_functor = identity_functor(m.category)
        and m.unit.dst_functor = m.functor
        and m.multiplication.src_functor = m.composite
        and m.multiplication.dst_functor = m.functor
        and forall(x: O) { monad_associativity_at(m.functor, m.multiplication, x) }
        and forall(x: O) { monad_unit_left_at(m.functor, m.unit, m.multiplication, x) }
        and forall(x: O) { monad_unit_right_at(m.functor, m.unit, m.multiplication, x) })
}

/// The unit of a monad is a natural transformation from the identity functor on the underlying category.
theorem monad_unit_src_functor[O, M](m: Monad[O, M]) {
    m.unit.src_functor = identity_functor(m.category)
} by {
    is_monad(m.category, m.functor, m.unit, m.multiplication, m.composite) =
        (m.functor.src_cat = m.category and m.functor.dst_cat = m.category
        and compose_functor(m.functor, m.functor) = Option.some(m.composite)
        and m.unit.src_functor = identity_functor(m.category)
        and m.unit.dst_functor = m.functor
        and m.multiplication.src_functor = m.composite
        and m.multiplication.dst_functor = m.functor
        and forall(x: O) { monad_associativity_at(m.functor, m.multiplication, x) }
        and forall(x: O) { monad_unit_left_at(m.functor, m.unit, m.multiplication, x) }
        and forall(x: O) { monad_unit_right_at(m.functor, m.unit, m.multiplication, x) })
}

/// The unit of a monad is a natural transformation to the endofunctor.
theorem monad_unit_dst_functor[O, M](m: Monad[O, M]) {
    m.unit.dst_functor = m.functor
} by {
    is_monad(m.category, m.functor, m.unit, m.multiplication, m.composite) =
        (m.functor.src_cat = m.category and m.functor.dst_cat = m.category
        and compose_functor(m.functor, m.functor) = Option.some(m.composite)
        and m.unit.src_functor = identity_functor(m.category)
        and m.unit.dst_functor = m.functor
        and m.multiplication.src_functor = m.composite
        and m.multiplication.dst_functor = m.functor
        and forall(x: O) { monad_associativity_at(m.functor, m.multiplication, x) }
        and forall(x: O) { monad_unit_left_at(m.functor, m.unit, m.multiplication, x) }
        and forall(x: O) { monad_unit_right_at(m.functor, m.unit, m.multiplication, x) })
}

/// The multiplication of a monad is a natural transformation from the composite endofunctor.
theorem monad_multiplication_src_functor[O, M](m: Monad[O, M]) {
    m.multiplication.src_functor = m.composite
} by {
    is_monad(m.category, m.functor, m.unit, m.multiplication, m.composite) =
        (m.functor.src_cat = m.category and m.functor.dst_cat = m.category
        and compose_functor(m.functor, m.functor) = Option.some(m.composite)
        and m.unit.src_functor = identity_functor(m.category)
        and m.unit.dst_functor = m.functor
        and m.multiplication.src_functor = m.composite
        and m.multiplication.dst_functor = m.functor
        and forall(x: O) { monad_associativity_at(m.functor, m.multiplication, x) }
        and forall(x: O) { monad_unit_left_at(m.functor, m.unit, m.multiplication, x) }
        and forall(x: O) { monad_unit_right_at(m.functor, m.unit, m.multiplication, x) })
}

/// The multiplication of a monad is a natural transformation to the endofunctor.
theorem monad_multiplication_dst_functor[O, M](m: Monad[O, M]) {
    m.multiplication.dst_functor = m.functor
} by {
    is_monad(m.category, m.functor, m.unit, m.multiplication, m.composite) =
        (m.functor.src_cat = m.category and m.functor.dst_cat = m.category
        and compose_functor(m.functor, m.functor) = Option.some(m.composite)
        and m.unit.src_functor = identity_functor(m.category)
        and m.unit.dst_functor = m.functor
        and m.multiplication.src_functor = m.composite
        and m.multiplication.dst_functor = m.functor
        and forall(x: O) { monad_associativity_at(m.functor, m.multiplication, x) }
        and forall(x: O) { monad_unit_left_at(m.functor, m.unit, m.multiplication, x) }
        and forall(x: O) { monad_unit_right_at(m.functor, m.unit, m.multiplication, x) })
}

/// The associativity law of a monad holds at every object.
theorem monad_associativity_all[O, M](m: Monad[O, M]) {
    forall(x: O) { monad_associativity_at(m.functor, m.multiplication, x) }
} by {
    is_monad(m.category, m.functor, m.unit, m.multiplication, m.composite) =
        (m.functor.src_cat = m.category and m.functor.dst_cat = m.category
        and compose_functor(m.functor, m.functor) = Option.some(m.composite)
        and m.unit.src_functor = identity_functor(m.category)
        and m.unit.dst_functor = m.functor
        and m.multiplication.src_functor = m.composite
        and m.multiplication.dst_functor = m.functor
        and forall(x: O) { monad_associativity_at(m.functor, m.multiplication, x) }
        and forall(x: O) { monad_unit_left_at(m.functor, m.unit, m.multiplication, x) }
        and forall(x: O) { monad_unit_right_at(m.functor, m.unit, m.multiplication, x) })
}

/// The left unit law of a monad holds at every object.
theorem monad_unit_left_all[O, M](m: Monad[O, M]) {
    forall(x: O) { monad_unit_left_at(m.functor, m.unit, m.multiplication, x) }
} by {
    is_monad(m.category, m.functor, m.unit, m.multiplication, m.composite) =
        (m.functor.src_cat = m.category and m.functor.dst_cat = m.category
        and compose_functor(m.functor, m.functor) = Option.some(m.composite)
        and m.unit.src_functor = identity_functor(m.category)
        and m.unit.dst_functor = m.functor
        and m.multiplication.src_functor = m.composite
        and m.multiplication.dst_functor = m.functor
        and forall(x: O) { monad_associativity_at(m.functor, m.multiplication, x) }
        and forall(x: O) { monad_unit_left_at(m.functor, m.unit, m.multiplication, x) }
        and forall(x: O) { monad_unit_right_at(m.functor, m.unit, m.multiplication, x) })
}

/// The right unit law of a monad holds at every object.
theorem monad_unit_right_all[O, M](m: Monad[O, M]) {
    forall(x: O) { monad_unit_right_at(m.functor, m.unit, m.multiplication, x) }
} by {
    is_monad(m.category, m.functor, m.unit, m.multiplication, m.composite) =
        (m.functor.src_cat = m.category and m.functor.dst_cat = m.category
        and compose_functor(m.functor, m.functor) = Option.some(m.composite)
        and m.unit.src_functor = identity_functor(m.category)
        and m.unit.dst_functor = m.functor
        and m.multiplication.src_functor = m.composite
        and m.multiplication.dst_functor = m.functor
        and forall(x: O) { monad_associativity_at(m.functor, m.multiplication, x) }
        and forall(x: O) { monad_unit_left_at(m.functor, m.unit, m.multiplication, x) }
        and forall(x: O) { monad_unit_right_at(m.functor, m.unit, m.multiplication, x) })
}

/// The associativity law of a monad, pointwise at the object `x`:
/// `mu_x ∘ t(mu_x) = mu_x ∘ mu_{t(x)}` in the target category of the endofunctor.
theorem monad_associativity_pointwise[O, M](m: Monad[O, M], x: O) {
    m.functor.dst_cat.compose(m.multiplication.component(x), m.functor.mor_map(m.multiplication.component(x)))
        = m.functor.dst_cat.compose(m.multiplication.component(x), m.multiplication.component(m.functor.obj_map(x)))
} by {
    monad_associativity_all(m)
    monad_associativity_at(m.functor, m.multiplication, x) = forall(y: O) {
        m.functor.dst_cat.compose(m.multiplication.component(y), m.functor.mor_map(m.multiplication.component(y)))
            = m.functor.dst_cat.compose(m.multiplication.component(y), m.multiplication.component(m.functor.obj_map(y)))
    }
}

/// The left unit law of a monad, pointwise at the object `x`:
/// `mu_x ∘ t(eta_x) = id_{t(x)}`.
theorem monad_unit_left_pointwise[O, M](m: Monad[O, M], x: O) {
    m.functor.dst_cat.compose(m.multiplication.component(x), m.functor.mor_map(m.unit.component(x)))
        = m.functor.dst_cat.identity(m.functor.obj_map(x))
} by {
    monad_unit_left_all(m)
    monad_unit_left_at(m.functor, m.unit, m.multiplication, x) = forall(y: O) {
        m.functor.dst_cat.compose(m.multiplication.component(y), m.functor.mor_map(m.unit.component(y)))
            = m.functor.dst_cat.identity(m.functor.obj_map(y))
    }
}

/// The right unit law of a monad, pointwise at the object `x`:
/// `mu_x ∘ eta_{t(x)} = id_{t(x)}`.
theorem monad_unit_right_pointwise[O, M](m: Monad[O, M], x: O) {
    m.functor.dst_cat.compose(m.multiplication.component(x), m.unit.component(m.functor.obj_map(x)))
        = m.functor.dst_cat.identity(m.functor.obj_map(x))
} by {
    monad_unit_right_all(m)
    monad_unit_right_at(m.functor, m.unit, m.multiplication, x) = forall(y: O) {
        m.functor.dst_cat.compose(m.multiplication.component(y), m.unit.component(m.functor.obj_map(y)))
            = m.functor.dst_cat.identity(m.functor.obj_map(y))
    }
}

/// The associativity law of a monad, pointwise in the underlying category.
theorem monad_associativity_cat[O, M](m: Monad[O, M], x: O) {
    m.category.compose(m.multiplication.component(x), m.functor.mor_map(m.multiplication.component(x)))
        = m.category.compose(m.multiplication.component(x), m.multiplication.component(m.functor.obj_map(x)))
} by {
    monad_associativity_pointwise(m, x)
    monad_functor_dst_cat(m)
}

/// The left unit law of a monad, pointwise in the underlying category.
theorem monad_unit_left_cat[O, M](m: Monad[O, M], x: O) {
    m.category.compose(m.multiplication.component(x), m.functor.mor_map(m.unit.component(x)))
        = m.category.identity(m.functor.obj_map(x))
} by {
    monad_unit_left_pointwise(m, x)
    monad_functor_dst_cat(m)
}

/// The right unit law of a monad, pointwise in the underlying category.
theorem monad_unit_right_cat[O, M](m: Monad[O, M], x: O) {
    m.category.compose(m.multiplication.component(x), m.unit.component(m.functor.obj_map(x)))
        = m.category.identity(m.functor.obj_map(x))
} by {
    monad_unit_right_pointwise(m, x)
    monad_functor_dst_cat(m)
}

/// The unit component `eta_x` is a morphism out of `x`.
theorem monad_unit_component_src[O, M](m: Monad[O, M], x: O) {
    m.category.src(m.unit.component(x)) = x
} by {
    monad_unit_src_functor(m)
    nat_trans_component_src(m.unit, x)
    identity_functor_src_cat(m.category)
    identity_functor_obj_map(m.category)
    m.category.src(m.unit.component(x)) = x
}

/// The unit component `eta_x` is a morphism into `t(x)`.
theorem monad_unit_component_dst[O, M](m: Monad[O, M], x: O) {
    m.category.dst(m.unit.component(x)) = m.functor.obj_map(x)
} by {
    monad_unit_dst_functor(m)
    nat_trans_component_dst(m.unit, x)
    monad_functor_dst_cat(m)
}

/// The multiplication component `mu_x` is a morphism out of `t(t(x))`.
theorem monad_multiplication_component_src[O, M](m: Monad[O, M], x: O) {
    m.category.src(m.multiplication.component(x)) = m.functor.obj_map(m.functor.obj_map(x))
} by {
    monad_multiplication_src_functor(m)
    nat_trans_component_src(m.multiplication, x)
    monad_functor_dst_cat(m)
    monad_compose(m)
    compose_functor_obj_map(m.functor, m.functor, m.composite)
    m.category.src(m.multiplication.component(x)) = m.functor.obj_map(m.functor.obj_map(x))
}

/// The multiplication component `mu_x` is a morphism into `t(x)`.
theorem monad_multiplication_component_dst[O, M](m: Monad[O, M], x: O) {
    m.category.dst(m.multiplication.component(x)) = m.functor.obj_map(x)
} by {
    monad_multiplication_dst_functor(m)
    nat_trans_component_dst(m.multiplication, x)
    monad_functor_dst_cat(m)
}

/// The unit is natural: `t(m) ∘ eta_x = eta_{x'} ∘ m` for `m: x -> x'`.
theorem monad_unit_naturality[O, M](m: Monad[O, M], p: M) {
    m.category.compose(
        m.functor.mor_map(p),
        m.unit.component(m.category.src(p))
    ) = m.category.compose(
        m.unit.component(m.category.dst(p)),
        p
    )
} by {
    nat_trans_naturality(m.unit, p)
    monad_unit_src_functor(m)
    monad_unit_dst_functor(m)
    monad_functor_src_cat(m)
    monad_functor_dst_cat(m)
    identity_functor_src_cat(m.category)
    identity_functor_dst_cat(m.category)
    identity_functor_mor_map(m.category)
}

/// The multiplication is natural: `mu_{x'} ∘ t²(m) = t(m) ∘ mu_x` for `m: x -> x'`.
theorem monad_multiplication_naturality[O, M](m: Monad[O, M], p: M) {
    m.category.compose(
        m.multiplication.component(m.category.dst(p)),
        m.functor.mor_map(m.functor.mor_map(p))
    ) = m.category.compose(
        m.functor.mor_map(p),
        m.multiplication.component(m.category.src(p))
    )
} by {
    nat_trans_naturality(m.multiplication, p)
    monad_multiplication_src_functor(m)
    monad_multiplication_dst_functor(m)
    monad_functor_src_cat(m)
    monad_functor_dst_cat(m)
    monad_compose(m)
    compose_functor_mor_map(m.functor, m.functor, m.composite)
    compose_functor_src_cat(m.functor, m.functor, m.composite)
    compose_functor_dst_cat(m.functor, m.functor, m.composite)
}

// ---------------------------------------------------------------------------
// Kleisli composition
// ---------------------------------------------------------------------------
//
// In the Kleisli category of a monad, a morphism from `A` to `B` is a
// morphism `f: A -> t(B)` of the underlying category. The Kleisli composite
// of `f: A -> t(B)` followed by `g: B -> t(C)` is
//   g ∘_K f = mu_C ∘ t(g) ∘ f,
// a morphism from `A` to `t(C)`. Kleisli composition is associative and has
// the unit components as identities; associativity is proved below.

/// The Kleisli composite of `f: A -> t(B)` followed by `g: B -> t(C)`:
/// `mu_C ∘ t(g) ∘ f`, a morphism from `A` to `t(C)`.
define kleisli_compose[O, M](m: Monad[O, M], z: O, f: M, g: M) -> M {
    m.category.compose(
        m.multiplication.component(z),
        m.category.compose(m.functor.mor_map(g), f))
}

/// The endofunctor of a monad preserves the Kleisli composite of `g` and `h`:
/// `t(h ∘_K g) = t(mu_D) ∘ t²(h) ∘ t(g)`.
theorem kleisli_compose_t_mor[O, M](
    m: Monad[O, M], y: O, z: O, w: O, g: M, h: M
) {
    m.category.src(g) = y and m.category.dst(g) = m.functor.obj_map(z)
    and m.category.src(h) = z and m.category.dst(h) = m.functor.obj_map(w)
    implies
    m.functor.mor_map(kleisli_compose(m, w, g, h))
        = m.category.compose(
            m.category.compose(
                m.functor.mor_map(m.multiplication.component(w)),
                m.functor.mor_map(m.functor.mor_map(h))),
            m.functor.mor_map(g))
} by {
    if m.category.src(g) = y and m.category.dst(g) = m.functor.obj_map(z)
        and m.category.src(h) = z and m.category.dst(h) = m.functor.obj_map(w) {
        monad_functor_src_cat(m)
        monad_functor_dst_cat(m)
        kleisli_compose(m, w, g, h) =
            m.category.compose(m.multiplication.component(w),
                m.category.compose(m.functor.mor_map(h), g))
        // composability of t(h) with g:
        functor_src(m.functor, h)
        functor_dst(m.functor, h)
        functor_src(m.functor, g)
        m.category.src(m.functor.mor_map(h)) = m.category.dst(g)
        // dst of t(h) is t(t(w)):
        m.category.dst(m.functor.mor_map(h)) = m.functor.obj_map(m.functor.obj_map(w))
        // src of mu(w) is t(t(w)):
        monad_multiplication_component_src(m, w)
        m.category.src(m.multiplication.component(w)) = m.functor.obj_map(m.functor.obj_map(w))
        // composable (mu(w), compose(t(h), g)):
        category_compose_dst(m.category, m.functor.mor_map(h), g)
        m.category.dst(m.category.compose(m.functor.mor_map(h), g)) =
            m.functor.obj_map(m.functor.obj_map(w))
        m.category.src(m.multiplication.component(w)) =
            m.category.dst(m.category.compose(m.functor.mor_map(h), g))
        m.functor.src_cat.src(m.multiplication.component(w)) =
            m.functor.src_cat.dst(m.category.compose(m.functor.mor_map(h), g))
        // t preserves the outer composition:
        functor_compose(m.functor, m.multiplication.component(w),
            m.category.compose(m.functor.mor_map(h), g))
        m.functor.mor_map(m.functor.src_cat.compose(m.multiplication.component(w),
            m.category.compose(m.functor.mor_map(h), g))) =
            m.functor.dst_cat.compose(m.functor.mor_map(m.multiplication.component(w)),
                m.functor.mor_map(m.category.compose(m.functor.mor_map(h), g)))
        m.functor.mor_map(m.category.compose(m.multiplication.component(w),
            m.category.compose(m.functor.mor_map(h), g))) =
            m.category.compose(m.functor.mor_map(m.multiplication.component(w)),
                m.functor.mor_map(m.category.compose(m.functor.mor_map(h), g)))
        // composability of t(h) with g, again for the inner composition:
        functor_compose(m.functor, m.functor.mor_map(h), g)
        m.functor.mor_map(m.functor.src_cat.compose(m.functor.mor_map(h), g)) =
            m.functor.dst_cat.compose(m.functor.mor_map(m.functor.mor_map(h)), m.functor.mor_map(g))
        m.functor.mor_map(m.category.compose(m.functor.mor_map(h), g)) =
            m.category.compose(m.functor.mor_map(m.functor.mor_map(h)), m.functor.mor_map(g))
        m.functor.mor_map(m.category.compose(m.multiplication.component(w),
            m.category.compose(m.functor.mor_map(h), g))) =
            m.category.compose(m.functor.mor_map(m.multiplication.component(w)),
                m.category.compose(m.functor.mor_map(m.functor.mor_map(h)), m.functor.mor_map(g)))
        m.functor.mor_map(kleisli_compose(m, w, g, h)) =
            m.category.compose(m.functor.mor_map(m.multiplication.component(w)),
                m.category.compose(m.functor.mor_map(m.functor.mor_map(h)), m.functor.mor_map(g)))
        // reassociate into the left-associated form:
        functor_src(m.functor, m.multiplication.component(w))
        monad_multiplication_component_src(m, w)
        m.category.src(m.functor.mor_map(m.multiplication.component(w))) =
            m.functor.obj_map(m.functor.obj_map(m.functor.obj_map(w)))
        functor_dst(m.functor, m.functor.mor_map(h))
        m.category.dst(m.functor.mor_map(m.functor.mor_map(h))) =
            m.functor.obj_map(m.functor.obj_map(m.functor.obj_map(w)))
        m.category.src(m.functor.mor_map(m.multiplication.component(w))) =
            m.category.dst(m.functor.mor_map(m.functor.mor_map(h)))
        functor_src(m.functor, m.functor.mor_map(h))
        m.category.src(m.functor.mor_map(m.functor.mor_map(h))) =
            m.functor.obj_map(m.functor.obj_map(z))
        functor_dst(m.functor, g)
        m.category.dst(m.functor.mor_map(g)) = m.functor.obj_map(m.functor.obj_map(z))
        m.category.src(m.functor.mor_map(m.functor.mor_map(h))) =
            m.category.dst(m.functor.mor_map(g))
        category_compose_assoc(m.category,
            m.functor.mor_map(m.multiplication.component(w)),
            m.functor.mor_map(m.functor.mor_map(h)),
            m.functor.mor_map(g))
        m.category.compose(
            m.functor.mor_map(m.multiplication.component(w)),
            m.category.compose(m.functor.mor_map(m.functor.mor_map(h)), m.functor.mor_map(g))) =
            m.category.compose(
                m.category.compose(
                    m.functor.mor_map(m.multiplication.component(w)),
                    m.functor.mor_map(m.functor.mor_map(h))),
                m.functor.mor_map(g))
        m.functor.mor_map(kleisli_compose(m, w, g, h)) =
            m.category.compose(
                m.category.compose(
                    m.functor.mor_map(m.multiplication.component(w)),
                    m.functor.mor_map(m.functor.mor_map(h))),
                m.functor.mor_map(g))
    }
}

/// The multiplication is natural at a morphism `p: z -> t(w)`:
/// `mu_{t(w)} ∘ t²(p) = t(p) ∘ mu_z`.
theorem monad_multiplication_naturality_at[O, M](
    m: Monad[O, M], z: O, w: O, p: M
) {
    m.category.src(p) = z and m.category.dst(p) = m.functor.obj_map(w) implies
    m.category.compose(
        m.multiplication.component(m.functor.obj_map(w)),
        m.functor.mor_map(m.functor.mor_map(p)))
        = m.category.compose(
            m.functor.mor_map(p),
            m.multiplication.component(z))
} by {
    if m.category.src(p) = z and m.category.dst(p) = m.functor.obj_map(w) {
        nat_trans_naturality(m.multiplication, p)
        monad_multiplication_src_functor(m)
        monad_multiplication_dst_functor(m)
        monad_functor_src_cat(m)
        monad_functor_dst_cat(m)
        monad_compose(m)
        compose_functor_mor_map(m.functor, m.functor, m.composite)
        compose_functor_src_cat(m.functor, m.functor, m.composite)
        compose_functor_dst_cat(m.functor, m.functor, m.composite)
        m.multiplication.src_functor.mor_map(p) =
            m.functor.mor_map(m.functor.mor_map(p))
        m.multiplication.component(m.category.dst(p)) =
            m.multiplication.component(m.functor.obj_map(w))
        m.multiplication.component(m.category.src(p)) = m.multiplication.component(z)
        m.category.compose(
            m.multiplication.component(m.category.dst(p)),
            m.functor.mor_map(m.functor.mor_map(p))) =
            m.category.compose(
                m.functor.mor_map(p),
                m.multiplication.component(m.category.src(p)))
    }
}

/// Kleisli composition is associative: `(h ∘_K g) ∘_K f = h ∘_K (g ∘_K f)`
/// for `f: x -> t(y)`, `g: y -> t(z)` and `h: z -> t(w)`.
theorem kleisli_compose_assoc[O, M](
    m: Monad[O, M], x: O, y: O, z: O, w: O, f: M, g: M, h: M
) {
    m.category.src(f) = x and m.category.dst(f) = m.functor.obj_map(y)
    and m.category.src(g) = y and m.category.dst(g) = m.functor.obj_map(z)
    and m.category.src(h) = z and m.category.dst(h) = m.functor.obj_map(w)
    implies
    kleisli_compose(m, w, f, kleisli_compose(m, w, g, h))
        = kleisli_compose(m, w, kleisli_compose(m, z, f, g), h)
} by {
    if m.category.src(f) = x and m.category.dst(f) = m.functor.obj_map(y)
        and m.category.src(g) = y and m.category.dst(g) = m.functor.obj_map(z)
        and m.category.src(h) = z and m.category.dst(h) = m.functor.obj_map(w) {
        monad_functor_src_cat(m)
        monad_functor_dst_cat(m)
        // unfold the left-hand side:
        kleisli_compose(m, w, f, kleisli_compose(m, w, g, h)) =
            m.category.compose(m.multiplication.component(w),
                m.category.compose(m.functor.mor_map(kleisli_compose(m, w, g, h)), f))
        // t preserves the Kleisli composite of g and h:
        kleisli_compose_t_mor(m, y, z, w, g, h)
        m.functor.mor_map(kleisli_compose(m, w, g, h)) =
            m.category.compose(
                m.category.compose(
                    m.functor.mor_map(m.multiplication.component(w)),
                    m.functor.mor_map(m.functor.mor_map(h))),
                m.functor.mor_map(g))
        kleisli_compose(m, w, f, kleisli_compose(m, w, g, h)) =
            m.category.compose(m.multiplication.component(w),
                m.category.compose(
                    m.category.compose(
                        m.category.compose(
                            m.functor.mor_map(m.multiplication.component(w)),
                            m.functor.mor_map(m.functor.mor_map(h))),
                        m.functor.mor_map(g)),
                    f))
        // endpoints of the pieces:
        functor_src(m.functor, h)
        functor_dst(m.functor, h)
        m.category.src(m.functor.mor_map(h)) = m.functor.obj_map(z)
        m.category.dst(m.functor.mor_map(h)) = m.functor.obj_map(m.functor.obj_map(w))
        functor_src(m.functor, m.functor.mor_map(h))
        functor_dst(m.functor, m.functor.mor_map(h))
        m.category.src(m.functor.mor_map(m.functor.mor_map(h))) =
            m.functor.obj_map(m.functor.obj_map(z))
        m.category.dst(m.functor.mor_map(m.functor.mor_map(h))) =
            m.functor.obj_map(m.functor.obj_map(m.functor.obj_map(w)))
        functor_src(m.functor, g)
        functor_dst(m.functor, g)
        m.category.src(m.functor.mor_map(g)) = m.functor.obj_map(y)
        m.category.dst(m.functor.mor_map(g)) = m.functor.obj_map(m.functor.obj_map(z))
        functor_src(m.functor, m.multiplication.component(w))
        functor_dst(m.functor, m.multiplication.component(w))
        monad_multiplication_component_src(m, w)
        monad_multiplication_component_dst(m, w)
        m.category.src(m.functor.mor_map(m.multiplication.component(w))) =
            m.functor.obj_map(m.functor.obj_map(m.functor.obj_map(w)))
        m.category.dst(m.functor.mor_map(m.multiplication.component(w))) =
            m.functor.obj_map(m.functor.obj_map(w))
        m.category.src(m.multiplication.component(w)) =
            m.functor.obj_map(m.functor.obj_map(w))
        m.category.dst(m.multiplication.component(w)) = m.functor.obj_map(w)
        monad_multiplication_component_src(m, m.functor.obj_map(w))
        monad_multiplication_component_dst(m, m.functor.obj_map(w))
        m.category.src(m.multiplication.component(m.functor.obj_map(w))) =
            m.functor.obj_map(m.functor.obj_map(m.functor.obj_map(w)))
        m.category.dst(m.multiplication.component(m.functor.obj_map(w))) =
            m.functor.obj_map(m.functor.obj_map(w))
        monad_multiplication_component_src(m, z)
        monad_multiplication_component_dst(m, z)
        m.category.src(m.multiplication.component(z)) =
            m.functor.obj_map(m.functor.obj_map(z))
        m.category.dst(m.multiplication.component(z)) = m.functor.obj_map(z)
        // step 1: reassociate (t(mu_w) ∘ t²(h)) ∘ (t(g) ∘ f)
        m.category.src(m.functor.mor_map(m.multiplication.component(w))) =
            m.category.dst(m.functor.mor_map(m.functor.mor_map(h)))
        category_compose_src(m.category,
            m.functor.mor_map(m.multiplication.component(w)),
            m.functor.mor_map(m.functor.mor_map(h)))
        m.category.src(m.category.compose(
            m.functor.mor_map(m.multiplication.component(w)),
            m.functor.mor_map(m.functor.mor_map(h)))) =
            m.category.src(m.functor.mor_map(m.functor.mor_map(h)))
        m.category.src(m.category.compose(
            m.functor.mor_map(m.multiplication.component(w)),
            m.functor.mor_map(m.functor.mor_map(h)))) =
            m.functor.obj_map(m.functor.obj_map(z))
        m.category.src(m.functor.mor_map(g)) = m.category.dst(f)
        category_compose_dst(m.category, m.functor.mor_map(g), f)
        m.category.dst(m.category.compose(m.functor.mor_map(g), f)) =
            m.category.dst(m.functor.mor_map(g))
        m.category.dst(m.category.compose(m.functor.mor_map(g), f)) =
            m.functor.obj_map(m.functor.obj_map(z))
        m.category.src(m.category.compose(
            m.functor.mor_map(m.multiplication.component(w)),
            m.functor.mor_map(m.functor.mor_map(h)))) =
            m.category.dst(m.category.compose(m.functor.mor_map(g), f))
        category_compose_assoc(m.category,
            m.category.compose(
                m.functor.mor_map(m.multiplication.component(w)),
                m.functor.mor_map(m.functor.mor_map(h))),
            m.functor.mor_map(g),
            f)
        m.category.compose(
            m.category.compose(
                m.category.compose(
                    m.functor.mor_map(m.multiplication.component(w)),
                    m.functor.mor_map(m.functor.mor_map(h))),
                m.functor.mor_map(g)),
            f) =
            m.category.compose(
                m.category.compose(
                    m.functor.mor_map(m.multiplication.component(w)),
                    m.functor.mor_map(m.functor.mor_map(h))),
                m.category.compose(m.functor.mor_map(g), f))
        kleisli_compose(m, w, f, kleisli_compose(m, w, g, h)) =
            m.category.compose(m.multiplication.component(w),
                m.category.compose(
                    m.category.compose(
                        m.functor.mor_map(m.multiplication.component(w)),
                        m.functor.mor_map(m.functor.mor_map(h))),
                    m.category.compose(m.functor.mor_map(g), f)))
        // step 2: reassociate (t(mu_w) ∘ t²(h)) ∘ (t(g) ∘ f) = t(mu_w) ∘ (t²(h) ∘ (t(g) ∘ f))
        m.category.src(m.functor.mor_map(m.multiplication.component(w))) =
            m.category.dst(m.functor.mor_map(m.functor.mor_map(h)))
        m.category.src(m.functor.mor_map(m.functor.mor_map(h))) =
            m.category.dst(m.category.compose(m.functor.mor_map(g), f))
        category_compose_assoc(m.category,
            m.functor.mor_map(m.multiplication.component(w)),
            m.functor.mor_map(m.functor.mor_map(h)),
            m.category.compose(m.functor.mor_map(g), f))
        m.category.compose(
            m.category.compose(
                m.functor.mor_map(m.multiplication.component(w)),
                m.functor.mor_map(m.functor.mor_map(h))),
            m.category.compose(m.functor.mor_map(g), f)) =
            m.category.compose(
                m.functor.mor_map(m.multiplication.component(w)),
                m.category.compose(
                    m.functor.mor_map(m.functor.mor_map(h)),
                    m.category.compose(m.functor.mor_map(g), f)))
        kleisli_compose(m, w, f, kleisli_compose(m, w, g, h)) =
            m.category.compose(m.multiplication.component(w),
                m.category.compose(
                    m.functor.mor_map(m.multiplication.component(w)),
                    m.category.compose(
                        m.functor.mor_map(m.functor.mor_map(h)),
                        m.category.compose(m.functor.mor_map(g), f))))
        // step 3: pull mu_w out: (mu_w ∘ t(mu_w)) ∘ (t²(h) ∘ (t(g) ∘ f))
        m.category.src(m.multiplication.component(w)) =
            m.category.dst(m.functor.mor_map(m.multiplication.component(w)))
        m.category.src(m.functor.mor_map(m.multiplication.component(w))) =
            m.category.dst(m.category.compose(
                m.functor.mor_map(m.functor.mor_map(h)),
                m.category.compose(m.functor.mor_map(g), f)))
        category_compose_assoc(m.category,
            m.multiplication.component(w),
            m.functor.mor_map(m.multiplication.component(w)),
            m.category.compose(
                m.functor.mor_map(m.functor.mor_map(h)),
                m.category.compose(m.functor.mor_map(g), f)))
        kleisli_compose(m, w, f, kleisli_compose(m, w, g, h)) =
            m.category.compose(
                m.category.compose(
                    m.multiplication.component(w),
                    m.functor.mor_map(m.multiplication.component(w))),
                m.category.compose(
                    m.functor.mor_map(m.functor.mor_map(h)),
                    m.category.compose(m.functor.mor_map(g), f)))
        // step 4: associativity law at w
        monad_associativity_cat(m, w)
        kleisli_compose(m, w, f, kleisli_compose(m, w, g, h)) =
            m.category.compose(
                m.category.compose(
                    m.multiplication.component(w),
                    m.multiplication.component(m.functor.obj_map(w))),
                m.category.compose(
                    m.functor.mor_map(m.functor.mor_map(h)),
                    m.category.compose(m.functor.mor_map(g), f)))
        // step 5: push mu_w back in
        m.category.src(m.multiplication.component(w)) =
            m.category.dst(m.multiplication.component(m.functor.obj_map(w)))
        m.category.src(m.multiplication.component(m.functor.obj_map(w))) =
            m.category.dst(m.category.compose(
                m.functor.mor_map(m.functor.mor_map(h)),
                m.category.compose(m.functor.mor_map(g), f)))
        category_compose_assoc(m.category,
            m.multiplication.component(w),
            m.multiplication.component(m.functor.obj_map(w)),
            m.category.compose(
                m.functor.mor_map(m.functor.mor_map(h)),
                m.category.compose(m.functor.mor_map(g), f)))
        kleisli_compose(m, w, f, kleisli_compose(m, w, g, h)) =
            m.category.compose(m.multiplication.component(w),
                m.category.compose(
                    m.multiplication.component(m.functor.obj_map(w)),
                    m.category.compose(
                        m.functor.mor_map(m.functor.mor_map(h)),
                        m.category.compose(m.functor.mor_map(g), f))))
        // step 6: reassociate mu_{t(w)} ∘ (t²(h) ∘ (t(g) ∘ f))
        m.category.src(m.multiplication.component(m.functor.obj_map(w))) =
            m.category.dst(m.functor.mor_map(m.functor.mor_map(h)))
        m.category.src(m.functor.mor_map(m.functor.mor_map(h))) =
            m.category.dst(m.category.compose(m.functor.mor_map(g), f))
        category_compose_assoc(m.category,
            m.multiplication.component(m.functor.obj_map(w)),
            m.functor.mor_map(m.functor.mor_map(h)),
            m.category.compose(m.functor.mor_map(g), f))
        m.category.compose(
            m.category.compose(
                m.multiplication.component(m.functor.obj_map(w)),
                m.functor.mor_map(m.functor.mor_map(h))),
            m.category.compose(m.functor.mor_map(g), f)) =
            m.category.compose(
                m.multiplication.component(m.functor.obj_map(w)),
                m.category.compose(
                    m.functor.mor_map(m.functor.mor_map(h)),
                    m.category.compose(m.functor.mor_map(g), f)))
        kleisli_compose(m, w, f, kleisli_compose(m, w, g, h)) =
            m.category.compose(m.multiplication.component(w),
                m.category.compose(
                    m.category.compose(
                        m.multiplication.component(m.functor.obj_map(w)),
                        m.functor.mor_map(m.functor.mor_map(h))),
                    m.category.compose(m.functor.mor_map(g), f)))
        // step 7: naturality of mu at h
        monad_multiplication_naturality_at(m, z, w, h)
        kleisli_compose(m, w, f, kleisli_compose(m, w, g, h)) =
            m.category.compose(m.multiplication.component(w),
                m.category.compose(
                    m.category.compose(
                        m.functor.mor_map(h),
                        m.multiplication.component(z)),
                    m.category.compose(m.functor.mor_map(g), f)))
        // step 8: reassociate (t(h) ∘ mu_z) ∘ (t(g) ∘ f)
        m.category.src(m.functor.mor_map(h)) =
            m.category.dst(m.multiplication.component(z))
        m.category.src(m.multiplication.component(z)) =
            m.category.dst(m.category.compose(m.functor.mor_map(g), f))
        category_compose_assoc(m.category,
            m.functor.mor_map(h),
            m.multiplication.component(z),
            m.category.compose(m.functor.mor_map(g), f))
        m.category.compose(
            m.category.compose(
                m.functor.mor_map(h),
                m.multiplication.component(z)),
            m.category.compose(m.functor.mor_map(g), f)) =
            m.category.compose(
                m.functor.mor_map(h),
                m.category.compose(
                    m.multiplication.component(z),
                    m.category.compose(m.functor.mor_map(g), f)))
        kleisli_compose(m, w, f, kleisli_compose(m, w, g, h)) =
            m.category.compose(m.multiplication.component(w),
                m.category.compose(
                    m.functor.mor_map(h),
                    m.category.compose(
                        m.multiplication.component(z),
                        m.category.compose(m.functor.mor_map(g), f))))
        // step 9: fold the inner Kleisli composite
        kleisli_compose(m, w, f, kleisli_compose(m, w, g, h)) =
            m.category.compose(m.multiplication.component(w),
                m.category.compose(m.functor.mor_map(h),
                    kleisli_compose(m, z, f, g)))
        // step 10: unfold the right-hand side
        kleisli_compose(m, w, f, kleisli_compose(m, w, g, h)) =
            kleisli_compose(m, w, kleisli_compose(m, z, f, g), h)
    }
}

/// The unit at `y` is a left identity for Kleisli composition:
/// `eta_y ∘_K f = f` for `f: x -> t(y)`.
theorem kleisli_compose_id_left[O, M](
    m: Monad[O, M], x: O, y: O, f: M
) {
    m.category.src(f) = x and m.category.dst(f) = m.functor.obj_map(y) implies
    kleisli_compose(m, y, f, m.unit.component(y)) = f
} by {
    if m.category.src(f) = x and m.category.dst(f) = m.functor.obj_map(y) {
        monad_functor_src_cat(m)
        monad_functor_dst_cat(m)
        kleisli_compose(m, y, f, m.unit.component(y)) =
            m.category.compose(m.multiplication.component(y),
                m.category.compose(m.functor.mor_map(m.unit.component(y)), f))
        // endpoints of t(eta_y) and mu_y:
        functor_src(m.functor, m.unit.component(y))
        functor_dst(m.functor, m.unit.component(y))
        monad_unit_component_src(m, y)
        monad_unit_component_dst(m, y)
        m.category.src(m.functor.mor_map(m.unit.component(y))) =
            m.functor.obj_map(y)
        m.category.dst(m.functor.mor_map(m.unit.component(y))) =
            m.functor.obj_map(m.functor.obj_map(y))
        m.category.src(m.functor.mor_map(m.unit.component(y))) =
            m.category.dst(f)
        monad_multiplication_component_src(m, y)
        m.category.src(m.multiplication.component(y)) =
            m.functor.obj_map(m.functor.obj_map(y))
        m.category.src(m.multiplication.component(y)) =
            m.category.dst(m.functor.mor_map(m.unit.component(y)))
        // reassociate: mu_y ∘ (t(eta_y) ∘ f) = (mu_y ∘ t(eta_y)) ∘ f
        category_compose_assoc(m.category,
            m.multiplication.component(y),
            m.functor.mor_map(m.unit.component(y)),
            f)
        m.category.compose(m.multiplication.component(y),
            m.category.compose(m.functor.mor_map(m.unit.component(y)), f)) =
            m.category.compose(
                m.category.compose(
                    m.multiplication.component(y),
                    m.functor.mor_map(m.unit.component(y))),
                f)
        // the left unit law of the monad at y:
        monad_unit_left_cat(m, y)
        m.category.compose(m.multiplication.component(y),
            m.functor.mor_map(m.unit.component(y))) =
            m.category.identity(m.functor.obj_map(y))
        m.category.compose(
            m.category.compose(
                m.multiplication.component(y),
                m.functor.mor_map(m.unit.component(y))),
            f) =
            m.category.compose(m.category.identity(m.functor.obj_map(y)), f)
        category_id_left(m.category, f)
        m.category.compose(m.category.identity(m.category.dst(f)), f) = f
        m.category.compose(m.category.identity(m.functor.obj_map(y)), f) = f
        m.category.compose(m.multiplication.component(y),
            m.category.compose(m.functor.mor_map(m.unit.component(y)), f)) = f
        kleisli_compose(m, y, f, m.unit.component(y)) = f
    }
}

/// The unit at `x` is a right identity for Kleisli composition:
/// `f ∘_K eta_x = f` for `f: x -> t(y)`.
theorem kleisli_compose_id_right[O, M](
    m: Monad[O, M], x: O, y: O, f: M
) {
    m.category.src(f) = x and m.category.dst(f) = m.functor.obj_map(y) implies
    kleisli_compose(m, y, m.unit.component(x), f) = f
} by {
    if m.category.src(f) = x and m.category.dst(f) = m.functor.obj_map(y) {
        monad_functor_src_cat(m)
        monad_functor_dst_cat(m)
        kleisli_compose(m, y, m.unit.component(x), f) =
            m.category.compose(m.multiplication.component(y),
                m.category.compose(m.functor.mor_map(f), m.unit.component(x)))
        // naturality of the unit at f: t(f) ∘ eta_x = eta_{t(y)} ∘ f
        monad_unit_naturality(m, f)
        m.category.compose(m.functor.mor_map(f), m.unit.component(x)) =
            m.category.compose(m.unit.component(m.functor.obj_map(y)), f)
        // endpoints:
        functor_src(m.functor, f)
        functor_dst(m.functor, f)
        m.category.src(m.functor.mor_map(f)) = m.functor.obj_map(x)
        m.category.dst(m.functor.mor_map(f)) = m.functor.obj_map(m.functor.obj_map(y))
        monad_unit_component_src(m, x)
        monad_unit_component_dst(m, x)
        monad_unit_component_src(m, m.functor.obj_map(y))
        monad_unit_component_dst(m, m.functor.obj_map(y))
        m.category.src(m.functor.mor_map(f)) = m.category.dst(m.unit.component(x))
        m.category.dst(m.category.compose(m.functor.mor_map(f), m.unit.component(x))) =
            m.functor.obj_map(m.functor.obj_map(y))
        monad_multiplication_component_src(m, y)
        m.category.src(m.multiplication.component(y)) =
            m.category.dst(m.category.compose(m.functor.mor_map(f), m.unit.component(x)))
        kleisli_compose(m, y, m.unit.component(x), f) =
            m.category.compose(m.multiplication.component(y),
                m.category.compose(m.unit.component(m.functor.obj_map(y)), f))
        // reassociate: mu_y ∘ (eta_{t(y)} ∘ f) = (mu_y ∘ eta_{t(y)}) ∘ f
        m.category.src(m.multiplication.component(y)) =
            m.category.dst(m.unit.component(m.functor.obj_map(y)))
        m.category.src(m.unit.component(m.functor.obj_map(y))) =
            m.category.dst(f)
        category_compose_assoc(m.category,
            m.multiplication.component(y),
            m.unit.component(m.functor.obj_map(y)),
            f)
        m.category.compose(m.multiplication.component(y),
            m.category.compose(m.unit.component(m.functor.obj_map(y)), f)) =
            m.category.compose(
                m.category.compose(
                    m.multiplication.component(y),
                    m.unit.component(m.functor.obj_map(y))),
                f)
        // the right unit law of the monad at y:
        monad_unit_right_cat(m, y)
        m.category.compose(m.multiplication.component(y),
            m.unit.component(m.functor.obj_map(y))) =
            m.category.identity(m.functor.obj_map(y))
        m.category.compose(
            m.category.compose(
                m.multiplication.component(y),
                m.unit.component(m.functor.obj_map(y))),
            f) =
            m.category.compose(m.category.identity(m.functor.obj_map(y)), f)
        category_id_left(m.category, f)
        m.category.compose(m.category.identity(m.category.dst(f)), f) = f
        m.category.compose(m.category.identity(m.functor.obj_map(y)), f) = f
        m.category.compose(m.multiplication.component(y),
            m.category.compose(m.unit.component(m.functor.obj_map(y)), f)) = f
        kleisli_compose(m, y, m.unit.component(x), f) = f
    }
}

// ---------------------------------------------------------------------------
// The monad induced by an adjunction (stated, not formalized)
// ---------------------------------------------------------------------------
//
// Every adjunction `a : Adjunction[O1, M1, O2, M2]` with left adjoint
// `f = a.left : C -> D` and right adjoint `g = a.right : D -> C` induces a
// monad on `C`:
//   t   = g ∘ f                       (the unit composite `a.unit_composite`)
//   eta = a.unit                      (the adjunction unit)
//   mu  = g(eps_f), the whiskering of the counit by `f` on the right and by
//         `g` on the left, whose component at an object `x` is
//         mu_x = g(eps_{f(x)}) : g(f(g(f(x)))) -> g(f(x)).
// The laws follow from the triangle identities and naturality of the counit:
//   associativity: g(eps_{f(x)}) ∘ g(f(g(eps_{f(x)})))
//                    = g(eps_{f(x)} ∘ f(g(eps_{f(x)})))             [functoriality of g]
//                    = g(eps_{f(x)} ∘ eps_{f(g(f(x)))})             [naturality of eps at eps_{f(x)}]
//                    = g(eps_{f(x)}) ∘ g(eps_{f(g(f(x)))})          [functoriality of g]
//                    = mu_x ∘ mu_{t(x)};
//   left unit:  mu_x ∘ t(eta_x) = g(eps_{f(x)} ∘ f(eta_x))
//                              = g(id_{f(x)})                       [left triangle identity]
//                              = id_{t(x)}                          [functoriality of g];
//   right unit: mu_x ∘ eta_{t(x)} = g(eps_{f(x)}) ∘ eta_{g(f(x))}
//                                = id_{g(f(x))}                     [right triangle identity at f(x)].
//
// Formalizing this requires building `mu` as a bundled natural transformation
// with source `t ∘ t = (g ∘ f) ∘ (g ∘ f)`. Using the whiskering API one would
// set
//   eps_f   = left_whisker_nat_trans(a.left, a.counit, fgf, f)      // eps ∘ f
//   mu      = right_whisker_nat_trans(eps_f, a.right, t2, t),       // g(eps_f)
// where `fgf = compose_functor(a.counit_composite, a.left)`,
// `f = compose_functor(identity_functor(d), a.left)` (which is `a.left` by
// `compose_functor_identity_left`), `t2 = compose_functor(a.unit_composite,
// a.unit_composite)` and `t = a.unit_composite`; the equality of the bundled
// composites with the monad's `functor` and `composite` fields then needs
// functor extensionality, and the three laws need the triangle identities
// `adjunction_triangle_left_at` and `adjunction_triangle_right_at` together
// with the naturality of the counit. The statement below records the goal:
//
// theorem adjunction_induced_monad[O1, M1, O2, M2](a: Adjunction[O1, M1, O2, M2]) {
//     exists(m: Monad[O1, M1]) {
//         m.category = a.left.src_cat
//         and m.functor = a.unit_composite
//         and m.unit = a.unit
//         and m.multiplication.component =
//             function(x: O1) { a.right.mor_map(a.counit.component(a.left.obj_map(x))) }
//     }
// }
