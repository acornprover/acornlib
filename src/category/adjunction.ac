/// Adjunctions between categories, in unit–counit form.
///
/// A unit–counit adjunction between functors `f: C -> D` and `g: D -> C` is a
/// pair of natural transformations
///   eta : id_C -> g ∘ f      (the unit)
///   eps  : f ∘ g -> id_D     (the counit)
/// satisfying the triangle identities
///   eps_{f(X)} ∘ f(eta_X) = id_{f(X)}          for every object X of C,
///   g(eps_Y) ∘ eta_{g(Y)} = id_{g(Y)}          for every object Y of D.
///
/// The equivalent hom-set formulation Hom_D(f(X), Y) ≅ Hom_C(X, g(Y)) is
/// expressed by the transpose maps below; the full statement would need hom
/// functors into the category of sets, which the library does not yet have.

from category.category import Category, category_id_left, category_id_right,
    category_compose_assoc, category_compose_src, category_compose_dst
from category.functor import Functor, identity_functor, identity_functor_src_cat,
    identity_functor_dst_cat, identity_functor_obj_map, identity_functor_mor_map,
    compose_functor, compose_functor_obj_map, compose_functor_mor_map,
    compose_functor_src_cat, compose_functor_dst_cat,
    functor_src, functor_dst, functor_compose
from category.natural_transformation import NaturalTransformation, nat_trans_naturality,
    nat_trans_component_src, nat_trans_component_dst,
    nat_trans_shared_src_cat, nat_trans_shared_dst_cat
from data.basic.functions import compose

/// True if the left triangle identity holds at every object:
/// `eps_{f(X)} ∘ f(eta_X) = id_{f(X)}` in the target category.
define adjunction_triangle_left[O1, M1, O2, M2](
    f: Functor[O1, M1, O2, M2],
    eta: NaturalTransformation[O1, M1, O1, M1],
    eps: NaturalTransformation[O2, M2, O2, M2]
) -> Bool {
    forall(x: O1) {
        f.dst_cat.compose(
            eps.component(f.obj_map(x)),
            f.mor_map(eta.component(x))
        ) = f.dst_cat.identity(f.obj_map(x))
    }
}

/// True if the right triangle identity holds at every object:
/// `g(eps_Y) ∘ eta_{g(Y)} = id_{g(Y)}` in the source category.
define adjunction_triangle_right[O1, M1, O2, M2](
    g: Functor[O2, M2, O1, M1],
    eta: NaturalTransformation[O1, M1, O1, M1],
    eps: NaturalTransformation[O2, M2, O2, M2]
) -> Bool {
    forall(y: O2) {
        g.dst_cat.compose(
            g.mor_map(eps.component(y)),
            eta.component(g.obj_map(y))
        ) = g.dst_cat.identity(g.obj_map(y))
    }
}

/// True if `eta` and `eps` form a unit–counit adjunction between `f: C -> D`
/// and `g: D -> C`, with `gf = g ∘ f` and `fg = f ∘ g` the composite functors.
define is_adjunction[O1, M1, O2, M2](
    c: Category[O1, M1],
    d: Category[O2, M2],
    f: Functor[O1, M1, O2, M2],
    g: Functor[O2, M2, O1, M1],
    eta: NaturalTransformation[O1, M1, O1, M1],
    eps: NaturalTransformation[O2, M2, O2, M2],
    gf: Functor[O1, M1, O1, M1],
    fg: Functor[O2, M2, O2, M2]
) -> Bool {
    compose_functor(g, f) = Option.some(gf)
    and compose_functor(f, g) = Option.some(fg)
    and eta.src_functor = identity_functor(c)
    and eta.dst_functor = gf
    and eps.src_functor = fg
    and eps.dst_functor = identity_functor(d)
    and adjunction_triangle_left(f, eta, eps)
    and adjunction_triangle_right(g, eta, eps)
}

/// An adjunction between `f` and `g` ensures `g ∘ f` is the chosen unit composite.
theorem adjunction_iso_left_compose[O1, M1, O2, M2](
    c: Category[O1, M1], d: Category[O2, M2],
    f: Functor[O1, M1, O2, M2], g: Functor[O2, M2, O1, M1],
    eta: NaturalTransformation[O1, M1, O1, M1],
    eps: NaturalTransformation[O2, M2, O2, M2],
    gf: Functor[O1, M1, O1, M1], fg: Functor[O2, M2, O2, M2]
) {
    is_adjunction(c, d, f, g, eta, eps, gf, fg) implies
        compose_functor(g, f) = Option.some(gf)
} by {
    is_adjunction(c, d, f, g, eta, eps, gf, fg) =
        (compose_functor(g, f) = Option.some(gf)
        and compose_functor(f, g) = Option.some(fg)
        and eta.src_functor = identity_functor(c)
        and eta.dst_functor = gf
        and eps.src_functor = fg
        and eps.dst_functor = identity_functor(d)
        and adjunction_triangle_left(f, eta, eps)
        and adjunction_triangle_right(g, eta, eps))
}

/// An adjunction between `f` and `g` ensures `f ∘ g` is the chosen counit composite.
theorem adjunction_iso_right_compose[O1, M1, O2, M2](
    c: Category[O1, M1], d: Category[O2, M2],
    f: Functor[O1, M1, O2, M2], g: Functor[O2, M2, O1, M1],
    eta: NaturalTransformation[O1, M1, O1, M1],
    eps: NaturalTransformation[O2, M2, O2, M2],
    gf: Functor[O1, M1, O1, M1], fg: Functor[O2, M2, O2, M2]
) {
    is_adjunction(c, d, f, g, eta, eps, gf, fg) implies
        compose_functor(f, g) = Option.some(fg)
} by {
    is_adjunction(c, d, f, g, eta, eps, gf, fg) =
        (compose_functor(g, f) = Option.some(gf)
        and compose_functor(f, g) = Option.some(fg)
        and eta.src_functor = identity_functor(c)
        and eta.dst_functor = gf
        and eps.src_functor = fg
        and eps.dst_functor = identity_functor(d)
        and adjunction_triangle_left(f, eta, eps)
        and adjunction_triangle_right(g, eta, eps))
}

/// The unit of an adjunction is a natural transformation from the identity functor on `c`.
theorem adjunction_iso_unit_src[O1, M1, O2, M2](
    c: Category[O1, M1], d: Category[O2, M2],
    f: Functor[O1, M1, O2, M2], g: Functor[O2, M2, O1, M1],
    eta: NaturalTransformation[O1, M1, O1, M1],
    eps: NaturalTransformation[O2, M2, O2, M2],
    gf: Functor[O1, M1, O1, M1], fg: Functor[O2, M2, O2, M2]
) {
    is_adjunction(c, d, f, g, eta, eps, gf, fg) implies
        eta.src_functor = identity_functor(c)
} by {
    is_adjunction(c, d, f, g, eta, eps, gf, fg) =
        (compose_functor(g, f) = Option.some(gf)
        and compose_functor(f, g) = Option.some(fg)
        and eta.src_functor = identity_functor(c)
        and eta.dst_functor = gf
        and eps.src_functor = fg
        and eps.dst_functor = identity_functor(d)
        and adjunction_triangle_left(f, eta, eps)
        and adjunction_triangle_right(g, eta, eps))
}

/// The unit of an adjunction has the chosen composite `g ∘ f` as target functor.
theorem adjunction_iso_unit_dst[O1, M1, O2, M2](
    c: Category[O1, M1], d: Category[O2, M2],
    f: Functor[O1, M1, O2, M2], g: Functor[O2, M2, O1, M1],
    eta: NaturalTransformation[O1, M1, O1, M1],
    eps: NaturalTransformation[O2, M2, O2, M2],
    gf: Functor[O1, M1, O1, M1], fg: Functor[O2, M2, O2, M2]
) {
    is_adjunction(c, d, f, g, eta, eps, gf, fg) implies
        eta.dst_functor = gf
} by {
    is_adjunction(c, d, f, g, eta, eps, gf, fg) =
        (compose_functor(g, f) = Option.some(gf)
        and compose_functor(f, g) = Option.some(fg)
        and eta.src_functor = identity_functor(c)
        and eta.dst_functor = gf
        and eps.src_functor = fg
        and eps.dst_functor = identity_functor(d)
        and adjunction_triangle_left(f, eta, eps)
        and adjunction_triangle_right(g, eta, eps))
}

/// The counit of an adjunction has the chosen composite `f ∘ g` as source functor.
theorem adjunction_iso_counit_src[O1, M1, O2, M2](
    c: Category[O1, M1], d: Category[O2, M2],
    f: Functor[O1, M1, O2, M2], g: Functor[O2, M2, O1, M1],
    eta: NaturalTransformation[O1, M1, O1, M1],
    eps: NaturalTransformation[O2, M2, O2, M2],
    gf: Functor[O1, M1, O1, M1], fg: Functor[O2, M2, O2, M2]
) {
    is_adjunction(c, d, f, g, eta, eps, gf, fg) implies
        eps.src_functor = fg
} by {
    is_adjunction(c, d, f, g, eta, eps, gf, fg) =
        (compose_functor(g, f) = Option.some(gf)
        and compose_functor(f, g) = Option.some(fg)
        and eta.src_functor = identity_functor(c)
        and eta.dst_functor = gf
        and eps.src_functor = fg
        and eps.dst_functor = identity_functor(d)
        and adjunction_triangle_left(f, eta, eps)
        and adjunction_triangle_right(g, eta, eps))
}

/// The counit of an adjunction is a natural transformation to the identity functor on `d`.
theorem adjunction_iso_counit_dst[O1, M1, O2, M2](
    c: Category[O1, M1], d: Category[O2, M2],
    f: Functor[O1, M1, O2, M2], g: Functor[O2, M2, O1, M1],
    eta: NaturalTransformation[O1, M1, O1, M1],
    eps: NaturalTransformation[O2, M2, O2, M2],
    gf: Functor[O1, M1, O1, M1], fg: Functor[O2, M2, O2, M2]
) {
    is_adjunction(c, d, f, g, eta, eps, gf, fg) implies
        eps.dst_functor = identity_functor(d)
} by {
    is_adjunction(c, d, f, g, eta, eps, gf, fg) =
        (compose_functor(g, f) = Option.some(gf)
        and compose_functor(f, g) = Option.some(fg)
        and eta.src_functor = identity_functor(c)
        and eta.dst_functor = gf
        and eps.src_functor = fg
        and eps.dst_functor = identity_functor(d)
        and adjunction_triangle_left(f, eta, eps)
        and adjunction_triangle_right(g, eta, eps))
}

/// An adjunction satisfies the left triangle identity, pointwise at every object.
theorem adjunction_iso_triangle_left[O1, M1, O2, M2](
    c: Category[O1, M1], d: Category[O2, M2],
    f: Functor[O1, M1, O2, M2], g: Functor[O2, M2, O1, M1],
    eta: NaturalTransformation[O1, M1, O1, M1],
    eps: NaturalTransformation[O2, M2, O2, M2],
    gf: Functor[O1, M1, O1, M1], fg: Functor[O2, M2, O2, M2]
) {
    is_adjunction(c, d, f, g, eta, eps, gf, fg) implies
        adjunction_triangle_left(f, eta, eps)
} by {
    is_adjunction(c, d, f, g, eta, eps, gf, fg) =
        (compose_functor(g, f) = Option.some(gf)
        and compose_functor(f, g) = Option.some(fg)
        and eta.src_functor = identity_functor(c)
        and eta.dst_functor = gf
        and eps.src_functor = fg
        and eps.dst_functor = identity_functor(d)
        and adjunction_triangle_left(f, eta, eps)
        and adjunction_triangle_right(g, eta, eps))
}

/// An adjunction satisfies the right triangle identity, pointwise at every object.
theorem adjunction_iso_triangle_right[O1, M1, O2, M2](
    c: Category[O1, M1], d: Category[O2, M2],
    f: Functor[O1, M1, O2, M2], g: Functor[O2, M2, O1, M1],
    eta: NaturalTransformation[O1, M1, O1, M1],
    eps: NaturalTransformation[O2, M2, O2, M2],
    gf: Functor[O1, M1, O1, M1], fg: Functor[O2, M2, O2, M2]
) {
    is_adjunction(c, d, f, g, eta, eps, gf, fg) implies
        adjunction_triangle_right(g, eta, eps)
} by {
    is_adjunction(c, d, f, g, eta, eps, gf, fg) =
        (compose_functor(g, f) = Option.some(gf)
        and compose_functor(f, g) = Option.some(fg)
        and eta.src_functor = identity_functor(c)
        and eta.dst_functor = gf
        and eps.src_functor = fg
        and eps.dst_functor = identity_functor(d)
        and adjunction_triangle_left(f, eta, eps)
        and adjunction_triangle_right(g, eta, eps))
}

/// The left triangle identity, pointwise: `eps_{f(X)} ∘ f(eta_X) = id_{f(X)}`.
theorem adjunction_triangle_left_apply[O1, M1, O2, M2](
    f: Functor[O1, M1, O2, M2],
    eta: NaturalTransformation[O1, M1, O1, M1],
    eps: NaturalTransformation[O2, M2, O2, M2],
    x: O1
) {
    adjunction_triangle_left(f, eta, eps) implies
        f.dst_cat.compose(
            eps.component(f.obj_map(x)),
            f.mor_map(eta.component(x))
        ) = f.dst_cat.identity(f.obj_map(x))
} by {
    adjunction_triangle_left(f, eta, eps) =
        forall(y: O1) {
            f.dst_cat.compose(
                eps.component(f.obj_map(y)),
                f.mor_map(eta.component(y))
            ) = f.dst_cat.identity(f.obj_map(y))
        }
}

/// The right triangle identity, pointwise: `g(eps_Y) ∘ eta_{g(Y)} = id_{g(Y)}`.
theorem adjunction_triangle_right_apply[O1, M1, O2, M2](
    g: Functor[O2, M2, O1, M1],
    eta: NaturalTransformation[O1, M1, O1, M1],
    eps: NaturalTransformation[O2, M2, O2, M2],
    y: O2
) {
    adjunction_triangle_right(g, eta, eps) implies
        g.dst_cat.compose(
            g.mor_map(eps.component(y)),
            eta.component(g.obj_map(y))
        ) = g.dst_cat.identity(g.obj_map(y))
} by {
    adjunction_triangle_right(g, eta, eps) =
        forall(z: O2) {
            g.dst_cat.compose(
                g.mor_map(eps.component(z)),
                eta.component(g.obj_map(z))
            ) = g.dst_cat.identity(g.obj_map(z))
        }
}

/// An adjunction between two categories, given by unit and counit.
/// `left: C -> D` is left adjoint to `right: D -> C`.
structure Adjunction[O1, M1, O2, M2] {
    /// The left adjoint functor from the source category to the target category.
    left: Functor[O1, M1, O2, M2]

    /// The right adjoint functor from the target category to the source category.
    right: Functor[O2, M2, O1, M1]

    /// The unit of the adjunction, a natural transformation from the identity on the source category to `right ∘ left`.
    unit: NaturalTransformation[O1, M1, O1, M1]

    /// The counit of the adjunction, a natural transformation from `left ∘ right` to the identity on the target category.
    counit: NaturalTransformation[O2, M2, O2, M2]

    /// The composite `right ∘ left` of the two functors.
    unit_composite: Functor[O1, M1, O1, M1]

    /// The composite `left ∘ right` of the two functors.
    counit_composite: Functor[O2, M2, O2, M2]
} constraint {
    is_adjunction(left.src_cat, left.dst_cat, left, right, unit, counit,
        unit_composite, counit_composite)
}

/// The data of an adjunction form a unit–counit adjunction, recorded as the
/// structure constraint; the extraction theorems below state its consequences.

/// The composite of the right adjoint after the left adjoint is the unit composite.
theorem adjunction_left_compose[O1, M1, O2, M2](a: Adjunction[O1, M1, O2, M2]) {
    compose_functor(a.right, a.left) = Option.some(a.unit_composite)
} by {
    adjunction_iso_left_compose(a.left.src_cat, a.left.dst_cat, a.left, a.right,
        a.unit, a.counit, a.unit_composite, a.counit_composite)
}

/// The composite of the left adjoint after the right adjoint is the counit composite.
theorem adjunction_right_compose[O1, M1, O2, M2](a: Adjunction[O1, M1, O2, M2]) {
    compose_functor(a.left, a.right) = Option.some(a.counit_composite)
} by {
    adjunction_iso_right_compose(a.left.src_cat, a.left.dst_cat, a.left, a.right,
        a.unit, a.counit, a.unit_composite, a.counit_composite)
}

/// The unit is a natural transformation from the identity functor on the source category.
theorem adjunction_unit_src_functor[O1, M1, O2, M2](a: Adjunction[O1, M1, O2, M2]) {
    a.unit.src_functor = identity_functor(a.left.src_cat)
} by {
    adjunction_iso_unit_src(a.left.src_cat, a.left.dst_cat, a.left, a.right,
        a.unit, a.counit, a.unit_composite, a.counit_composite)
}

/// The unit is a natural transformation to the composite `right ∘ left`.
theorem adjunction_unit_dst_functor[O1, M1, O2, M2](a: Adjunction[O1, M1, O2, M2]) {
    a.unit.dst_functor = a.unit_composite
} by {
    adjunction_iso_unit_dst(a.left.src_cat, a.left.dst_cat, a.left, a.right,
        a.unit, a.counit, a.unit_composite, a.counit_composite)
}

/// The counit is a natural transformation from the composite `left ∘ right`.
theorem adjunction_counit_src_functor[O1, M1, O2, M2](a: Adjunction[O1, M1, O2, M2]) {
    a.counit.src_functor = a.counit_composite
} by {
    adjunction_iso_counit_src(a.left.src_cat, a.left.dst_cat, a.left, a.right,
        a.unit, a.counit, a.unit_composite, a.counit_composite)
}

/// The counit is a natural transformation to the identity functor on the target category.
theorem adjunction_counit_dst_functor[O1, M1, O2, M2](a: Adjunction[O1, M1, O2, M2]) {
    a.counit.dst_functor = identity_functor(a.left.dst_cat)
} by {
    adjunction_iso_counit_dst(a.left.src_cat, a.left.dst_cat, a.left, a.right,
        a.unit, a.counit, a.unit_composite, a.counit_composite)
}

/// The left triangle identity: `eps_{f(X)} ∘ f(eta_X) = id_{f(X)}` at every object `X`.
theorem adjunction_triangle_left_at[O1, M1, O2, M2](a: Adjunction[O1, M1, O2, M2], x: O1) {
    a.left.dst_cat.compose(
        a.counit.component(a.left.obj_map(x)),
        a.left.mor_map(a.unit.component(x))
    ) = a.left.dst_cat.identity(a.left.obj_map(x))
} by {
    adjunction_iso_triangle_left(a.left.src_cat, a.left.dst_cat, a.left, a.right,
        a.unit, a.counit, a.unit_composite, a.counit_composite)
    adjunction_triangle_left_apply(a.left, a.unit, a.counit, x)
}

/// The right triangle identity: `g(eps_Y) ∘ eta_{g(Y)} = id_{g(Y)}` at every object `Y`.
theorem adjunction_triangle_right_at[O1, M1, O2, M2](a: Adjunction[O1, M1, O2, M2], y: O2) {
    a.right.dst_cat.compose(
        a.right.mor_map(a.counit.component(y)),
        a.unit.component(a.right.obj_map(y))
    ) = a.right.dst_cat.identity(a.right.obj_map(y))
} by {
    adjunction_iso_triangle_right(a.left.src_cat, a.left.dst_cat, a.left, a.right,
        a.unit, a.counit, a.unit_composite, a.counit_composite)
    adjunction_triangle_right_apply(a.right, a.unit, a.counit, y)
}

/// The unit component `eta_X` is a morphism out of `X`.
theorem adjunction_unit_component_src[O1, M1, O2, M2](a: Adjunction[O1, M1, O2, M2], x: O1) {
    a.left.src_cat.src(a.unit.component(x)) = x
} by {
    adjunction_unit_src_functor(a)
    nat_trans_component_src(a.unit, x)
    identity_functor_src_cat(a.left.src_cat)
    identity_functor_obj_map(a.left.src_cat)
    a.left.src_cat.src(a.unit.component(x)) = x
}

/// The unit component `eta_X` is a morphism into `g(f(X))`.
theorem adjunction_unit_component_dst[O1, M1, O2, M2](a: Adjunction[O1, M1, O2, M2], x: O1) {
    a.left.src_cat.dst(a.unit.component(x)) = a.right.obj_map(a.left.obj_map(x))
} by {
    adjunction_unit_dst_functor(a)
    nat_trans_component_dst(a.unit, x)
    adjunction_left_compose(a)
    compose_functor_obj_map(a.right, a.left, a.unit_composite)
    a.left.src_cat.dst(a.unit.component(x)) = a.right.obj_map(a.left.obj_map(x))
}

/// The counit component `eps_Y` is a morphism out of `f(g(Y))`.
theorem adjunction_counit_component_src[O1, M1, O2, M2](a: Adjunction[O1, M1, O2, M2], y: O2) {
    a.left.dst_cat.src(a.counit.component(y)) = a.left.obj_map(a.right.obj_map(y))
} by {
    adjunction_counit_src_functor(a)
    nat_trans_component_src(a.counit, y)
    a.counit.src_functor.dst_cat.src(a.counit.component(y)) = a.counit.src_functor.obj_map(y)
    adjunction_right_compose(a)
    compose_functor(a.left, a.right) = Option.some(a.counit_composite)
    compose_functor_dst_cat(a.left, a.right, a.counit_composite)
    a.counit_composite.dst_cat = a.left.dst_cat
    compose_functor_obj_map(a.left, a.right, a.counit_composite)
    a.counit_composite.obj_map = compose(a.left.obj_map, a.right.obj_map)
    a.counit_composite.obj_map(y) = a.left.obj_map(a.right.obj_map(y))
    a.counit_composite.dst_cat.src(a.counit.component(y)) = a.counit_composite.obj_map(y)
    a.left.dst_cat.src(a.counit.component(y)) = a.left.obj_map(a.right.obj_map(y))
}

/// The counit component `eps_Y` is a morphism into `Y`.
theorem adjunction_counit_component_dst[O1, M1, O2, M2](a: Adjunction[O1, M1, O2, M2], y: O2) {
    a.left.dst_cat.dst(a.counit.component(y)) = y
} by {
    adjunction_counit_dst_functor(a)
    nat_trans_component_dst(a.counit, y)
    identity_functor_dst_cat(a.left.dst_cat)
    identity_functor_obj_map(a.left.dst_cat)
    a.left.dst_cat.dst(a.counit.component(y)) = y
}

/// The unit is natural: `(g∘f)(m) ∘ eta_X = eta_{X'} ∘ m` for `m: X -> X'`.
theorem adjunction_unit_naturality[O1, M1, O2, M2](a: Adjunction[O1, M1, O2, M2], m: M1) {
    a.left.src_cat.compose(
        a.unit_composite.mor_map(m),
        a.unit.component(a.left.src_cat.src(m))
    ) = a.left.src_cat.compose(
        a.unit.component(a.left.src_cat.dst(m)),
        m
    )
} by {
    nat_trans_naturality(a.unit, m)
    adjunction_unit_src_functor(a)
    adjunction_unit_dst_functor(a)
    identity_functor_src_cat(a.left.src_cat)
    identity_functor_dst_cat(a.left.src_cat)
    identity_functor_mor_map(a.left.src_cat)
}

/// The counit is natural: `eps_{Y'} ∘ (f∘g)(n) = n ∘ eps_Y` for `n: Y -> Y'`.
theorem adjunction_counit_naturality[O1, M1, O2, M2](a: Adjunction[O1, M1, O2, M2], n: M2) {
    a.left.dst_cat.compose(
        a.counit.component(a.left.dst_cat.dst(n)),
        a.counit_composite.mor_map(n)
    ) = a.left.dst_cat.compose(
        n,
        a.counit.component(a.left.dst_cat.src(n))
    )
} by {
    nat_trans_naturality(a.counit, n)
    nat_trans_shared_src_cat(a.counit)
    nat_trans_shared_dst_cat(a.counit)
    adjunction_counit_src_functor(a)
    adjunction_counit_dst_functor(a)
    identity_functor_src_cat(a.left.dst_cat)
    identity_functor_dst_cat(a.left.dst_cat)
    identity_functor_mor_map(a.left.dst_cat)
}

/// The target category of the right adjoint is the source category of the left adjoint.
theorem adjunction_right_dst_cat[O1, M1, O2, M2](a: Adjunction[O1, M1, O2, M2]) {
    a.right.dst_cat = a.left.src_cat
} by {
    adjunction_unit_src_functor(a)
    identity_functor_dst_cat(a.left.src_cat)
    a.unit.src_functor.dst_cat = a.left.src_cat
    adjunction_unit_dst_functor(a)
    a.unit.dst_functor.dst_cat = a.unit_composite.dst_cat
    adjunction_left_compose(a)
    compose_functor_dst_cat(a.right, a.left, a.unit_composite)
    a.unit_composite.dst_cat = a.right.dst_cat
    nat_trans_shared_dst_cat(a.unit)
    a.unit.src_functor.dst_cat = a.unit.dst_functor.dst_cat
    a.left.src_cat = a.right.dst_cat
    a.right.dst_cat = a.left.src_cat
}

/// The source category of the right adjoint is the target category of the left adjoint.
theorem adjunction_right_src_cat[O1, M1, O2, M2](a: Adjunction[O1, M1, O2, M2]) {
    a.right.src_cat = a.left.dst_cat
} by {
    adjunction_counit_src_functor(a)
    a.counit.src_functor.src_cat = a.counit_composite.src_cat
    adjunction_right_compose(a)
    compose_functor_src_cat(a.left, a.right, a.counit_composite)
    a.counit_composite.src_cat = a.right.src_cat
    adjunction_counit_dst_functor(a)
    identity_functor_src_cat(a.left.dst_cat)
    a.counit.dst_functor.src_cat = a.left.dst_cat
    nat_trans_shared_src_cat(a.counit)
    a.counit.src_functor.src_cat = a.counit.dst_functor.src_cat
    a.right.src_cat = a.left.dst_cat
}

// ---------------------------------------------------------------------------
// Transposes between hom-sets
// ---------------------------------------------------------------------------
//
// The adjunction gives a natural bijection Hom_D(f(X), Y) ≅ Hom_C(X, g(Y)):
// a morphism f: f(X) -> Y transposes to g(f) ∘ eta_X : X -> g(Y), and a
// morphism g: X -> g(Y) transposes to eps_Y ∘ f(g) : f(X) -> Y. The two
// round-trip theorems below show these transposes are inverse to each other,
// which is the hom-set form of the triangle identities.

/// The transpose of a morphism `f: left(X) -> Y` is `right(f) ∘ unit_X : X -> right(Y)`.
define adjunction_transpose_right[O1, M1, O2, M2](
    a: Adjunction[O1, M1, O2, M2], x: O1, y: O2, f: M2
) -> M1 {
    a.right.dst_cat.compose(a.right.mor_map(f), a.unit.component(x))
}

/// The transpose of a morphism `g: X -> right(Y)` is `counit_Y ∘ left(g) : left(X) -> Y`.
define adjunction_transpose_left[O1, M1, O2, M2](
    a: Adjunction[O1, M1, O2, M2], x: O1, y: O2, g: M1
) -> M2 {
    a.left.dst_cat.compose(a.counit.component(y), a.left.mor_map(g))
}

/// The transpose of `f: left(X) -> Y` has source `X`.
theorem adjunction_transpose_right_src[O1, M1, O2, M2](
    a: Adjunction[O1, M1, O2, M2], x: O1, y: O2, f: M2
) {
    a.left.dst_cat.src(f) = a.left.obj_map(x) implies
        a.left.src_cat.src(adjunction_transpose_right(a, x, y, f)) = x
} by {
    adjunction_right_dst_cat(a)
    adjunction_right_src_cat(a)
    functor_src(a.right, f)
    a.right.dst_cat.src(a.right.mor_map(f)) = a.right.obj_map(a.right.src_cat.src(f))
    a.left.src_cat.src(a.right.mor_map(f)) = a.right.obj_map(a.left.dst_cat.src(f))
    a.left.src_cat.src(a.right.mor_map(f)) = a.right.obj_map(a.left.obj_map(x))
    adjunction_unit_component_src(a, x)
    a.left.src_cat.src(a.unit.component(x)) = x
    adjunction_unit_component_dst(a, x)
    a.left.src_cat.dst(a.unit.component(x)) = a.right.obj_map(a.left.obj_map(x))
    a.left.src_cat.src(a.right.mor_map(f)) = a.left.src_cat.dst(a.unit.component(x))
    category_compose_src(a.left.src_cat, a.right.mor_map(f), a.unit.component(x))
    a.left.src_cat.src(
        a.left.src_cat.compose(a.right.mor_map(f), a.unit.component(x))) =
        a.left.src_cat.src(a.unit.component(x))
    a.left.src_cat.src(
        a.left.src_cat.compose(a.right.mor_map(f), a.unit.component(x))) = x
}

/// The transpose of `f: left(X) -> Y` has target `right(Y)`.
theorem adjunction_transpose_right_dst[O1, M1, O2, M2](
    a: Adjunction[O1, M1, O2, M2], x: O1, y: O2, f: M2
) {
    a.left.dst_cat.src(f) = a.left.obj_map(x) and a.left.dst_cat.dst(f) = y implies
        a.left.src_cat.dst(adjunction_transpose_right(a, x, y, f)) = a.right.obj_map(y)
} by {
    adjunction_right_dst_cat(a)
    adjunction_right_src_cat(a)
    functor_src(a.right, f)
    functor_dst(a.right, f)
    a.right.dst_cat.dst(a.right.mor_map(f)) = a.right.obj_map(a.right.src_cat.dst(f))
    a.left.src_cat.dst(a.right.mor_map(f)) = a.right.obj_map(a.left.dst_cat.dst(f))
    a.left.src_cat.dst(a.right.mor_map(f)) = a.right.obj_map(y)
    adjunction_unit_component_src(a, x)
    adjunction_unit_component_dst(a, x)
    a.left.src_cat.src(a.right.mor_map(f)) = a.right.obj_map(a.left.obj_map(x))
    a.left.src_cat.dst(a.unit.component(x)) = a.right.obj_map(a.left.obj_map(x))
    a.left.src_cat.src(a.right.mor_map(f)) = a.left.src_cat.dst(a.unit.component(x))
    category_compose_dst(a.left.src_cat, a.right.mor_map(f), a.unit.component(x))
    a.left.src_cat.dst(
        a.left.src_cat.compose(a.right.mor_map(f), a.unit.component(x))) =
        a.left.src_cat.dst(a.right.mor_map(f))
    a.left.src_cat.dst(
        a.left.src_cat.compose(a.right.mor_map(f), a.unit.component(x))) = a.right.obj_map(y)
}

/// The transpose of `g: X -> right(Y)` has source `left(X)`.
theorem adjunction_transpose_left_src[O1, M1, O2, M2](
    a: Adjunction[O1, M1, O2, M2], x: O1, y: O2, g: M1
) {
    a.left.src_cat.src(g) = x and a.left.src_cat.dst(g) = a.right.obj_map(y) implies
        a.left.dst_cat.src(adjunction_transpose_left(a, x, y, g)) = a.left.obj_map(x)
} by {
    functor_src(a.left, g)
    functor_dst(a.left, g)
    a.left.dst_cat.src(a.left.mor_map(g)) = a.left.obj_map(a.left.src_cat.src(g))
    a.left.dst_cat.src(a.left.mor_map(g)) = a.left.obj_map(x)
    a.left.dst_cat.dst(a.left.mor_map(g)) = a.left.obj_map(a.left.src_cat.dst(g))
    a.left.dst_cat.dst(a.left.mor_map(g)) = a.left.obj_map(a.right.obj_map(y))
    adjunction_counit_component_src(a, y)
    a.left.dst_cat.src(a.counit.component(y)) = a.left.obj_map(a.right.obj_map(y))
    a.left.dst_cat.src(a.counit.component(y)) = a.left.dst_cat.dst(a.left.mor_map(g))
    category_compose_src(a.left.dst_cat, a.counit.component(y), a.left.mor_map(g))
    a.left.dst_cat.src(
        a.left.dst_cat.compose(a.counit.component(y), a.left.mor_map(g))) =
        a.left.dst_cat.src(a.left.mor_map(g))
    a.left.dst_cat.src(
        a.left.dst_cat.compose(a.counit.component(y), a.left.mor_map(g))) = a.left.obj_map(x)
}


/// The transposes are inverse: transposing a morphism `f: left(X) -> Y` to the
/// right adjoint side and back recovers `f`. Together with the companion
/// theorem below this makes `Hom_D(left(X), Y) ≅ Hom_C(X, right(Y))` a bijection.
theorem adjunction_transpose_roundtrip_to_left[O1, M1, O2, M2](
    a: Adjunction[O1, M1, O2, M2], x: O1, y: O2, f: M2
) {
    a.left.dst_cat.src(f) = a.left.obj_map(x) and a.left.dst_cat.dst(f) = y implies
        adjunction_transpose_left(a, x, y, adjunction_transpose_right(a, x, y, f)) = f
} by {
    adjunction_right_dst_cat(a)
    adjunction_right_src_cat(a)
    functor_src(a.right, f)
    a.left.src_cat.src(a.right.mor_map(f)) = a.right.obj_map(a.left.dst_cat.src(f))
    a.left.src_cat.src(a.right.mor_map(f)) = a.right.obj_map(a.left.obj_map(x))
    adjunction_unit_component_dst(a, x)
    a.left.src_cat.dst(a.unit.component(x)) = a.right.obj_map(a.left.obj_map(x))
    a.left.src_cat.src(a.right.mor_map(f)) = a.left.src_cat.dst(a.unit.component(x))
    functor_compose(a.left, a.right.mor_map(f), a.unit.component(x))
    a.left.mor_map(a.left.src_cat.compose(a.right.mor_map(f), a.unit.component(x))) =
        a.left.dst_cat.compose(
            a.left.mor_map(a.right.mor_map(f)),
            a.left.mor_map(a.unit.component(x)))
    adjunction_right_compose(a)
    compose_functor_mor_map(a.left, a.right, a.counit_composite)
    a.left.mor_map(a.right.mor_map(f)) = a.counit_composite.mor_map(f)
    adjunction_counit_component_src(a, y)
    a.left.dst_cat.src(a.counit.component(y)) = a.left.obj_map(a.right.obj_map(y))
    functor_dst(a.right, f)
    functor_dst(a.left, a.right.mor_map(f))
    a.left.dst_cat.dst(a.left.mor_map(a.right.mor_map(f))) =
        a.left.obj_map(a.left.src_cat.dst(a.right.mor_map(f)))
    a.left.src_cat.dst(a.right.mor_map(f)) = a.right.obj_map(a.left.dst_cat.dst(f))
    a.left.dst_cat.dst(a.left.mor_map(a.right.mor_map(f))) = a.left.obj_map(a.right.obj_map(y))
    a.left.dst_cat.src(a.counit.component(y)) = a.left.dst_cat.dst(a.left.mor_map(a.right.mor_map(f)))
    functor_src(a.left, a.right.mor_map(f))
    functor_src(a.right, f)
    a.left.dst_cat.src(a.left.mor_map(a.right.mor_map(f))) =
        a.left.obj_map(a.left.src_cat.src(a.right.mor_map(f)))
    a.left.src_cat.src(a.right.mor_map(f)) = a.right.obj_map(a.left.dst_cat.src(f))
    a.left.dst_cat.src(a.left.mor_map(a.right.mor_map(f))) = a.left.obj_map(a.right.obj_map(a.left.obj_map(x)))
    functor_dst(a.left, a.unit.component(x))
    a.left.dst_cat.dst(a.left.mor_map(a.unit.component(x))) =
        a.left.obj_map(a.left.src_cat.dst(a.unit.component(x)))
    adjunction_unit_component_dst(a, x)
    a.left.src_cat.dst(a.unit.component(x)) = a.right.obj_map(a.left.obj_map(x))
    a.left.dst_cat.dst(a.left.mor_map(a.unit.component(x))) = a.left.obj_map(a.right.obj_map(a.left.obj_map(x)))
    a.left.dst_cat.src(a.left.mor_map(a.right.mor_map(f))) =
        a.left.dst_cat.dst(a.left.mor_map(a.unit.component(x)))
    category_compose_assoc(a.left.dst_cat, a.counit.component(y),
        a.left.mor_map(a.right.mor_map(f)), a.left.mor_map(a.unit.component(x)))
    a.left.dst_cat.compose(
        a.counit.component(y),
        a.left.dst_cat.compose(
            a.left.mor_map(a.right.mor_map(f)),
            a.left.mor_map(a.unit.component(x)))) =
        a.left.dst_cat.compose(
            a.left.dst_cat.compose(
                a.counit.component(y),
                a.left.mor_map(a.right.mor_map(f))),
            a.left.mor_map(a.unit.component(x)))
    adjunction_counit_naturality(a, f)
    a.left.dst_cat.dst(f) = y
    a.left.dst_cat.src(f) = a.left.obj_map(x)
    a.left.dst_cat.compose(a.counit.component(y), a.counit_composite.mor_map(f)) =
        a.left.dst_cat.compose(f, a.counit.component(a.left.obj_map(x)))
    adjunction_counit_component_dst(a, a.left.obj_map(x))
    a.left.dst_cat.dst(a.counit.component(a.left.obj_map(x))) = a.left.obj_map(x)
    a.left.dst_cat.src(f) = a.left.dst_cat.dst(a.counit.component(a.left.obj_map(x)))
    adjunction_counit_component_src(a, a.left.obj_map(x))
    a.left.dst_cat.src(a.counit.component(a.left.obj_map(x))) =
        a.left.obj_map(a.right.obj_map(a.left.obj_map(x)))
    functor_dst(a.left, a.unit.component(x))
    a.left.dst_cat.dst(a.left.mor_map(a.unit.component(x))) =
        a.left.obj_map(a.left.src_cat.dst(a.unit.component(x)))
    a.left.src_cat.dst(a.unit.component(x)) = a.right.obj_map(a.left.obj_map(x))
    a.left.dst_cat.dst(a.left.mor_map(a.unit.component(x))) = a.left.obj_map(a.right.obj_map(a.left.obj_map(x)))
    a.left.dst_cat.src(a.counit.component(a.left.obj_map(x))) =
        a.left.dst_cat.dst(a.left.mor_map(a.unit.component(x)))
    category_compose_assoc(a.left.dst_cat, f, a.counit.component(a.left.obj_map(x)),
        a.left.mor_map(a.unit.component(x)))
    a.left.dst_cat.compose(
        a.left.dst_cat.compose(f, a.counit.component(a.left.obj_map(x))),
        a.left.mor_map(a.unit.component(x))) =
        a.left.dst_cat.compose(
            f,
            a.left.dst_cat.compose(
                a.counit.component(a.left.obj_map(x)),
                a.left.mor_map(a.unit.component(x))))
    adjunction_triangle_left_at(a, x)
    a.left.dst_cat.compose(
        a.counit.component(a.left.obj_map(x)),
        a.left.mor_map(a.unit.component(x))) = a.left.dst_cat.identity(a.left.obj_map(x))
    a.left.dst_cat.src(f) = a.left.obj_map(x)
    category_id_right(a.left.dst_cat, f)
    a.left.dst_cat.compose(f, a.left.dst_cat.identity(a.left.obj_map(x))) = f
    adjunction_transpose_left(a, x, y, adjunction_transpose_right(a, x, y, f)) = f
}

/// The transposes are inverse: transposing a morphism `g: X -> right(Y)` to the
/// left adjoint side and back recovers `g`.
theorem adjunction_transpose_roundtrip_to_right[O1, M1, O2, M2](
    a: Adjunction[O1, M1, O2, M2], x: O1, y: O2, g: M1
) {
    a.left.src_cat.src(g) = x and a.left.src_cat.dst(g) = a.right.obj_map(y) implies
        adjunction_transpose_right(a, x, y, adjunction_transpose_left(a, x, y, g)) = g
} by {
    adjunction_counit_component_src(a, y)
    a.left.dst_cat.src(a.counit.component(y)) = a.left.obj_map(a.right.obj_map(y))
    functor_dst(a.left, g)
    a.left.dst_cat.dst(a.left.mor_map(g)) = a.left.obj_map(a.left.src_cat.dst(g))
    a.left.dst_cat.dst(a.left.mor_map(g)) = a.left.obj_map(a.right.obj_map(y))
    a.left.dst_cat.src(a.counit.component(y)) = a.left.dst_cat.dst(a.left.mor_map(g))
    adjunction_right_src_cat(a)
    a.right.src_cat.src(a.counit.component(y)) = a.right.src_cat.dst(a.left.mor_map(g))
    functor_compose(a.right, a.counit.component(y), a.left.mor_map(g))
    a.right.mor_map(a.right.src_cat.compose(a.counit.component(y), a.left.mor_map(g))) =
        a.right.dst_cat.compose(
            a.right.mor_map(a.counit.component(y)),
            a.right.mor_map(a.left.mor_map(g)))
    adjunction_right_dst_cat(a)
    a.right.mor_map(a.left.dst_cat.compose(a.counit.component(y), a.left.mor_map(g))) =
        a.left.src_cat.compose(
            a.right.mor_map(a.counit.component(y)),
            a.right.mor_map(a.left.mor_map(g)))
    adjunction_left_compose(a)
    compose_functor_mor_map(a.right, a.left, a.unit_composite)
    a.right.mor_map(a.left.mor_map(g)) = a.unit_composite.mor_map(g)
    functor_src(a.right, a.counit.component(y))
    a.left.src_cat.src(a.right.mor_map(a.counit.component(y))) =
        a.right.obj_map(a.left.dst_cat.src(a.counit.component(y)))
    a.left.src_cat.src(a.right.mor_map(a.counit.component(y))) =
        a.right.obj_map(a.left.obj_map(a.right.obj_map(y)))
    functor_dst(a.right, a.left.mor_map(g))
    a.left.src_cat.dst(a.right.mor_map(a.left.mor_map(g))) =
        a.right.obj_map(a.left.dst_cat.dst(a.left.mor_map(g)))
    a.left.src_cat.dst(a.right.mor_map(a.left.mor_map(g))) =
        a.right.obj_map(a.left.obj_map(a.right.obj_map(y)))
    a.left.src_cat.src(a.right.mor_map(a.counit.component(y))) =
        a.left.src_cat.dst(a.right.mor_map(a.left.mor_map(g)))
    functor_src(a.right, a.left.mor_map(g))
    functor_src(a.left, g)
    a.left.src_cat.src(a.right.mor_map(a.left.mor_map(g))) =
        a.right.obj_map(a.left.dst_cat.src(a.left.mor_map(g)))
    a.left.dst_cat.src(a.left.mor_map(g)) = a.left.obj_map(a.left.src_cat.src(g))
    a.left.src_cat.src(a.right.mor_map(a.left.mor_map(g))) = a.right.obj_map(a.left.obj_map(x))
    adjunction_unit_component_dst(a, x)
    a.left.src_cat.dst(a.unit.component(x)) = a.right.obj_map(a.left.obj_map(x))
    a.left.src_cat.src(a.right.mor_map(a.left.mor_map(g))) =
        a.left.src_cat.dst(a.unit.component(x))
    category_compose_assoc(a.left.src_cat, a.right.mor_map(a.counit.component(y)),
        a.right.mor_map(a.left.mor_map(g)), a.unit.component(x))
    a.left.src_cat.compose(
        a.left.src_cat.compose(
            a.right.mor_map(a.counit.component(y)),
            a.right.mor_map(a.left.mor_map(g))),
        a.unit.component(x)) =
        a.left.src_cat.compose(
            a.right.mor_map(a.counit.component(y)),
            a.left.src_cat.compose(
                a.right.mor_map(a.left.mor_map(g)),
                a.unit.component(x)))
    adjunction_unit_naturality(a, g)
    a.left.src_cat.src(g) = x
    a.left.src_cat.dst(g) = a.right.obj_map(y)
    a.left.src_cat.compose(
        a.unit_composite.mor_map(g),
        a.unit.component(x)) =
        a.left.src_cat.compose(
            a.unit.component(a.right.obj_map(y)),
            g)
    functor_src(a.right, a.counit.component(y))
    a.left.src_cat.src(a.right.mor_map(a.counit.component(y))) =
        a.right.obj_map(a.left.dst_cat.src(a.counit.component(y)))
    a.left.src_cat.src(a.right.mor_map(a.counit.component(y))) =
        a.right.obj_map(a.left.obj_map(a.right.obj_map(y)))
    adjunction_unit_component_dst(a, a.right.obj_map(y))
    a.left.src_cat.dst(a.unit.component(a.right.obj_map(y))) =
        a.right.obj_map(a.left.obj_map(a.right.obj_map(y)))
    a.left.src_cat.src(a.right.mor_map(a.counit.component(y))) =
        a.left.src_cat.dst(a.unit.component(a.right.obj_map(y)))
    adjunction_unit_component_src(a, a.right.obj_map(y))
    a.left.src_cat.src(a.unit.component(a.right.obj_map(y))) = a.right.obj_map(y)
    a.left.src_cat.src(a.unit.component(a.right.obj_map(y))) = a.left.src_cat.dst(g)
    category_compose_assoc(a.left.src_cat, a.right.mor_map(a.counit.component(y)),
        a.unit.component(a.right.obj_map(y)), g)
    a.left.src_cat.compose(
        a.right.mor_map(a.counit.component(y)),
        a.left.src_cat.compose(a.unit.component(a.right.obj_map(y)), g)) =
        a.left.src_cat.compose(
            a.left.src_cat.compose(
                a.right.mor_map(a.counit.component(y)),
                a.unit.component(a.right.obj_map(y))),
            g)
    adjunction_triangle_right_at(a, y)
    a.left.src_cat.compose(
        a.right.mor_map(a.counit.component(y)),
        a.unit.component(a.right.obj_map(y))) =
        a.left.src_cat.identity(a.right.obj_map(y))
    a.left.src_cat.dst(g) = a.right.obj_map(y)
    category_id_left(a.left.src_cat, g)
    a.left.src_cat.compose(a.left.src_cat.identity(a.right.obj_map(y)), g) = g
    adjunction_transpose_right(a, x, y, adjunction_transpose_left(a, x, y, g)) = g
}

/// The transpose of `g: X -> right(Y)` has target `Y`.
theorem adjunction_transpose_left_dst[O1, M1, O2, M2](
    a: Adjunction[O1, M1, O2, M2], x: O1, y: O2, g: M1
) {
    a.left.src_cat.dst(g) = a.right.obj_map(y) implies
        a.left.dst_cat.dst(adjunction_transpose_left(a, x, y, g)) = y
} by {
    functor_dst(a.left, g)
    a.left.dst_cat.dst(a.left.mor_map(g)) = a.left.obj_map(a.left.src_cat.dst(g))
    a.left.dst_cat.dst(a.left.mor_map(g)) = a.left.obj_map(a.right.obj_map(y))
    adjunction_counit_component_src(a, y)
    a.left.dst_cat.src(a.counit.component(y)) = a.left.obj_map(a.right.obj_map(y))
    a.left.dst_cat.src(a.counit.component(y)) = a.left.dst_cat.dst(a.left.mor_map(g))
    adjunction_counit_component_dst(a, y)
    a.left.dst_cat.dst(a.counit.component(y)) = y
    category_compose_dst(a.left.dst_cat, a.counit.component(y), a.left.mor_map(g))
    a.left.dst_cat.dst(
        a.left.dst_cat.compose(a.counit.component(y), a.left.mor_map(g))) =
        a.left.dst_cat.dst(a.counit.component(y))
    a.left.dst_cat.dst(
        a.left.dst_cat.compose(a.counit.component(y), a.left.mor_map(g))) = y
}

// ---------------------------------------------------------------------------
// Classic example (stated, not formalized)
// ---------------------------------------------------------------------------
//
// In a cartesian closed category, the product functor `_ × A` is left adjoint
// to the internal hom functor `[A, _]`:
//   unit_X : X -> [A, X × A]        (currying the identity on X × A)
//   counit_Y : [A, Y] × A -> Y      (evaluation)
// The triangle identities are exactly beta-reduction and eta-expansion. The
// library does not yet have a cartesian closed category (internal hom objects,
// evaluation and currying morphisms), so the statement is left as a comment.
