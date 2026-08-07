/// The discrete category over a type, where the only morphisms are identities.

from category.category import Category, is_category, category_id_endpoints_constraint,
    category_compose_src_constraint, category_compose_dst_constraint,
    category_assoc_constraint, category_id_left_constraint,
    category_id_right_constraint, category_new_src, category_new_dst,
    category_new_identity, category_new_compose
from data.basic.functions import identity_fn

/// Composition operator for the discrete category: returns its first argument.
/// This is well-defined on composable pairs because composability forces the
/// arguments to be equal.
define discrete_compose[O](f: O, g: O) -> O {
    f
}

/// The proposed data for the discrete category satisfies the category axioms.
theorem discrete_is_category[O](anchor: O) {
    is_category[O, O](identity_fn[O], identity_fn[O], identity_fn[O], discrete_compose[O])
} by {
    forall(x: O) {
        identity_fn[O](identity_fn[O](x)) = x
    }
    category_id_endpoints_constraint(identity_fn[O], identity_fn[O], identity_fn[O])
    forall(f: O, g: O) {
        if identity_fn[O](f) = identity_fn[O](g) {
            identity_fn[O](discrete_compose(f, g)) = identity_fn[O](g)
        }
    }
    category_compose_src_constraint(identity_fn[O], identity_fn[O], discrete_compose[O])
    forall(f: O, g: O) {
        if identity_fn[O](f) = identity_fn[O](g) {
            identity_fn[O](discrete_compose(f, g)) = identity_fn[O](f)
        }
    }
    category_compose_dst_constraint(identity_fn[O], identity_fn[O], discrete_compose[O])
    forall(f: O, g: O, h: O) {
        if identity_fn[O](f) = identity_fn[O](g) and identity_fn[O](g) = identity_fn[O](h) {
            discrete_compose(discrete_compose(f, g), h) = discrete_compose(f, discrete_compose(g, h))
        }
    }
    category_assoc_constraint(identity_fn[O], identity_fn[O], discrete_compose[O])
    forall(f: O) {
        discrete_compose(identity_fn[O](identity_fn[O](f)), f) = f
    }
    category_id_left_constraint(identity_fn[O], identity_fn[O], identity_fn[O], discrete_compose[O])
    forall(f: O) {
        discrete_compose(f, identity_fn[O](identity_fn[O](f))) = f
    }
    category_id_right_constraint(identity_fn[O], identity_fn[O], identity_fn[O], discrete_compose[O])
}

/// The discrete category over `O`.
let discrete_category[O](anchor: O) -> result: Category[O, O] satisfy {
    Category[O, O].new(identity_fn[O], identity_fn[O], identity_fn[O], discrete_compose[O])
        = Option.some(result)
} by {
    discrete_is_category(anchor)
}

/// The source map of the discrete category is the identity function.
theorem discrete_category_src[O](anchor: O) {
    discrete_category(anchor).src = identity_fn[O]
} by {
    let c = discrete_category(anchor)
    (Category[O, O].new(identity_fn[O], identity_fn[O], identity_fn[O], discrete_compose[O])
        = Option.some(c))
    category_new_src(identity_fn[O], identity_fn[O], identity_fn[O], discrete_compose[O], c)
}

/// The target map of the discrete category is the identity function.
theorem discrete_category_dst[O](anchor: O) {
    discrete_category(anchor).dst = identity_fn[O]
} by {
    let c = discrete_category(anchor)
    (Category[O, O].new(identity_fn[O], identity_fn[O], identity_fn[O], discrete_compose[O])
        = Option.some(c))
    category_new_dst(identity_fn[O], identity_fn[O], identity_fn[O], discrete_compose[O], c)
}

/// The identity-assigning map of the discrete category is the identity function.
theorem discrete_category_identity[O](anchor: O) {
    discrete_category(anchor).identity = identity_fn[O]
} by {
    let c = discrete_category(anchor)
    (Category[O, O].new(identity_fn[O], identity_fn[O], identity_fn[O], discrete_compose[O])
        = Option.some(c))
    category_new_identity(identity_fn[O], identity_fn[O], identity_fn[O], discrete_compose[O], c)
}

/// The composition operator of the discrete category is `discrete_compose`.
theorem discrete_category_compose[O](anchor: O) {
    discrete_category(anchor).compose = discrete_compose[O]
} by {
    let c = discrete_category(anchor)
    (Category[O, O].new(identity_fn[O], identity_fn[O], identity_fn[O], discrete_compose[O])
        = Option.some(c))
    category_new_compose(identity_fn[O], identity_fn[O], identity_fn[O], discrete_compose[O], c)
}

/// In a discrete category, source evaluates to the morphism itself.
theorem discrete_category_src_apply[O](anchor: O, x: O) {
    discrete_category(anchor).src(x) = x
} by {
    discrete_category_src(anchor)
}

/// In a discrete category, target evaluates to the morphism itself.
theorem discrete_category_dst_apply[O](anchor: O, x: O) {
    discrete_category(anchor).dst(x) = x
} by {
    discrete_category_dst(anchor)
}

/// In a discrete category, the identity morphism at an object is that object.
theorem discrete_category_identity_apply[O](anchor: O, x: O) {
    discrete_category(anchor).identity(x) = x
} by {
    discrete_category_identity(anchor)
}

/// In a discrete category, composition returns its first argument.
theorem discrete_category_compose_apply[O](anchor: O, f: O, g: O) {
    discrete_category(anchor).compose(f, g) = f
} by {
    discrete_category_compose(anchor)
}
