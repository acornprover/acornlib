/// The opposite category obtained by reversing every morphism.

from category.category import Category, is_category, category_id_endpoints_constraint,
    category_compose_src_constraint, category_compose_dst_constraint,
    category_assoc_constraint, category_id_left_constraint,
    category_id_right_constraint, category_identity_src, category_identity_dst,
    category_compose_src, category_compose_dst, category_compose_assoc,
    category_id_left, category_id_right, category_new_src, category_new_dst,
    category_new_identity, category_new_compose, category_ext

/// The reversed composition of a category: `opposite_compose(c, f, g) = c.compose(g, f)`.
define opposite_compose[O, M](c: Category[O, M], f: M, g: M) -> M {
    c.compose(g, f)
}

/// The reversed src/dst/compose data of a category satisfies the category axioms.
theorem opposite_is_category[O, M](c: Category[O, M]) {
    is_category[O, M](c.dst, c.src, c.identity, opposite_compose(c))
} by {
    forall(x: O) {
        category_identity_src(c, x)
        c.src(c.identity(x)) = x
        category_identity_dst(c, x)
        c.dst(c.identity(x)) = x
    }
    category_id_endpoints_constraint(c.dst, c.src, c.identity)
    forall(f: M, g: M) {
        if c.dst(f) = c.src(g) {
            opposite_compose(c, f, g) = c.compose(g, f)
            c.src(g) = c.dst(f)
            category_compose_src(c, g, f)
            c.src(c.compose(g, f)) = c.src(f)
            c.src(opposite_compose(c, f, g)) = c.src(f)
        }
    }
    category_compose_src_constraint(c.dst, c.src, opposite_compose(c))
    forall(f: M, g: M) {
        if c.dst(f) = c.src(g) {
            opposite_compose(c, f, g) = c.compose(g, f)
            c.src(g) = c.dst(f)
            category_compose_dst(c, g, f)
            c.dst(c.compose(g, f)) = c.dst(g)
            c.dst(opposite_compose(c, f, g)) = c.dst(g)
        }
    }
    category_compose_dst_constraint(c.dst, c.src, opposite_compose(c))
    forall(f: M, g: M, h: M) {
        if c.dst(f) = c.src(g) and c.dst(g) = c.src(h) {
            opposite_compose(c, f, g) = c.compose(g, f)
            opposite_compose(c, g, h) = c.compose(h, g)
            opposite_compose(c, opposite_compose(c, f, g), h) = c.compose(h, c.compose(g, f))
            opposite_compose(c, f, opposite_compose(c, g, h)) = c.compose(c.compose(h, g), f)
            c.src(h) = c.dst(g)
            c.src(g) = c.dst(f)
            category_compose_assoc(c, h, g, f)
            c.compose(c.compose(h, g), f) = c.compose(h, c.compose(g, f))
            opposite_compose(c, opposite_compose(c, f, g), h) = opposite_compose(c, f, opposite_compose(c, g, h))
        }
    }
    category_assoc_constraint(c.dst, c.src, opposite_compose(c))
    forall(f: M) {
        opposite_compose(c, c.identity(c.src(f)), f) = c.compose(f, c.identity(c.src(f)))
        category_id_right(c, f)
        c.compose(f, c.identity(c.src(f))) = f
        opposite_compose(c, c.identity(c.src(f)), f) = f
    }
    category_id_left_constraint(c.dst, c.src, c.identity, opposite_compose(c))
    forall(f: M) {
        opposite_compose(c, f, c.identity(c.dst(f))) = c.compose(c.identity(c.dst(f)), f)
        category_id_left(c, f)
        c.compose(c.identity(c.dst(f)), f) = f
        opposite_compose(c, f, c.identity(c.dst(f))) = f
    }
    category_id_right_constraint(c.dst, c.src, c.identity, opposite_compose(c))
}

/// The opposite category obtained by reversing every morphism in `c`.
let opposite_category[O, M](c: Category[O, M]) -> result: Category[O, M] satisfy {
    Category[O, M].new(c.dst, c.src, c.identity, opposite_compose(c)) = Option.some(result)
} by {
    opposite_is_category(c)
}

/// The source map of the opposite category is the original target map.
theorem opposite_category_src[O, M](c: Category[O, M]) {
    opposite_category(c).src = c.dst
} by {
    let d = opposite_category(c)
    (Category[O, M].new(c.dst, c.src, c.identity, opposite_compose(c)) = Option.some(d))
    category_new_src(c.dst, c.src, c.identity, opposite_compose(c), d)
}

/// The target map of the opposite category is the original source map.
theorem opposite_category_dst[O, M](c: Category[O, M]) {
    opposite_category(c).dst = c.src
} by {
    let d = opposite_category(c)
    (Category[O, M].new(c.dst, c.src, c.identity, opposite_compose(c)) = Option.some(d))
    category_new_dst(c.dst, c.src, c.identity, opposite_compose(c), d)
}

/// The identity-assigning map of the opposite category is the original identity-assigning map.
theorem opposite_category_identity[O, M](c: Category[O, M]) {
    opposite_category(c).identity = c.identity
} by {
    let d = opposite_category(c)
    (Category[O, M].new(c.dst, c.src, c.identity, opposite_compose(c)) = Option.some(d))
    category_new_identity(c.dst, c.src, c.identity, opposite_compose(c), d)
}

/// The composition operator of the opposite category is `opposite_compose`.
theorem opposite_category_compose[O, M](c: Category[O, M]) {
    opposite_category(c).compose = opposite_compose(c)
} by {
    let d = opposite_category(c)
    (Category[O, M].new(c.dst, c.src, c.identity, opposite_compose(c)) = Option.some(d))
    category_new_compose(c.dst, c.src, c.identity, opposite_compose(c), d)
}

/// Taking the opposite category twice recovers the original category.
theorem opposite_category_involutive[O, M](c: Category[O, M]) {
    opposite_category(opposite_category(c)) = c
} by {
    let d = opposite_category(c)
    opposite_category_src(c)
    opposite_category_dst(c)
    opposite_category_identity(c)
    opposite_category_compose(c)
    d.src = c.dst
    d.dst = c.src
    d.identity = c.identity
    d.compose = opposite_compose(c)
    let e = opposite_category(d)
    opposite_category_src(d)
    opposite_category_dst(d)
    opposite_category_identity(d)
    opposite_category_compose(d)
    e.src = d.dst
    e.dst = d.src
    e.identity = d.identity
    e.compose = opposite_compose(d)
    forall(m: M) {
        e.src(m) = c.src(m)
    }
    forall(m: M) {
        e.dst(m) = c.dst(m)
    }
    forall(x: O) {
        e.identity(x) = c.identity(x)
    }
    forall(f: M, g: M) {
        e.compose(f, g) = opposite_compose(d, f, g)
        opposite_compose(d, f, g) = d.compose(g, f)
        d.compose(g, f) = opposite_compose(c, g, f)
        opposite_compose(c, g, f) = c.compose(f, g)
        e.compose(f, g) = c.compose(f, g)
    }
    category_ext(e, c)
}
