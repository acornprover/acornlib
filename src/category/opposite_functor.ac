/// Functors and natural transformations over opposite categories.

from category.category import Category
from category.opposite_category import opposite_category, opposite_compose,
    opposite_category_src, opposite_category_dst, opposite_category_identity,
    opposite_category_compose, opposite_category_involutive
from category.functor import Functor, is_functor, functor_src_axiom, functor_dst_axiom,
    functor_identity_axiom, functor_compose_axiom, functor_src, functor_dst,
    functor_identity, functor_compose, functor_new_src_cat, functor_new_dst_cat,
    functor_new_obj_map, functor_new_mor_map, functor_ext, identity_functor,
    identity_functor_src_cat, identity_functor_dst_cat, identity_functor_obj_map,
    identity_functor_mor_map
from category.natural_transformation import NaturalTransformation, functors_parallel,
    nat_trans_endpoints_axiom, nat_trans_naturality_axiom,
    is_natural_transformation, nat_trans_shared_src_cat, nat_trans_shared_dst_cat,
    nat_trans_component_src, nat_trans_component_dst, nat_trans_naturality,
    nat_trans_new_src_functor, nat_trans_new_dst_functor, nat_trans_new_component,
    nat_trans_ext
from data.basic.functions import identity_fn

/// The same object and morphism maps satisfy the source axiom on opposite categories.
theorem opposite_functor_src_axiom[O1, M1, O2, M2](f: Functor[O1, M1, O2, M2]) {
    functor_src_axiom(opposite_category(f.src_cat), opposite_category(f.dst_cat),
        f.obj_map, f.mor_map)
} by {
    forall(m: M1) {
        opposite_category_src(f.dst_cat)
        opposite_category(f.dst_cat).src = f.dst_cat.dst
        functor_dst(f, m)
        f.dst_cat.dst(f.mor_map(m)) = f.obj_map(f.src_cat.dst(m))
        opposite_category_src(f.src_cat)
        opposite_category(f.src_cat).src = f.src_cat.dst
        f.obj_map(opposite_category(f.src_cat).src(m)) = f.obj_map(f.src_cat.dst(m))
        opposite_category(f.dst_cat).src(f.mor_map(m)) =
            f.obj_map(opposite_category(f.src_cat).src(m))
    }
}

/// The same object and morphism maps satisfy the target axiom on opposite categories.
theorem opposite_functor_dst_axiom[O1, M1, O2, M2](f: Functor[O1, M1, O2, M2]) {
    functor_dst_axiom(opposite_category(f.src_cat), opposite_category(f.dst_cat),
        f.obj_map, f.mor_map)
} by {
    forall(m: M1) {
        opposite_category_dst(f.dst_cat)
        opposite_category(f.dst_cat).dst = f.dst_cat.src
        functor_src(f, m)
        f.dst_cat.src(f.mor_map(m)) = f.obj_map(f.src_cat.src(m))
        opposite_category_dst(f.src_cat)
        opposite_category(f.src_cat).dst = f.src_cat.src
        f.obj_map(opposite_category(f.src_cat).dst(m)) = f.obj_map(f.src_cat.src(m))
        opposite_category(f.dst_cat).dst(f.mor_map(m)) =
            f.obj_map(opposite_category(f.src_cat).dst(m))
    }
}

/// The same maps preserve identities on opposite categories.
theorem opposite_functor_identity_axiom[O1, M1, O2, M2](f: Functor[O1, M1, O2, M2]) {
    functor_identity_axiom(opposite_category(f.src_cat), opposite_category(f.dst_cat),
        f.obj_map, f.mor_map)
} by {
    forall(x: O1) {
        opposite_category_identity(f.src_cat)
        opposite_category(f.src_cat).identity = f.src_cat.identity
        opposite_category_identity(f.dst_cat)
        opposite_category(f.dst_cat).identity = f.dst_cat.identity
        functor_identity(f, x)
        f.mor_map(f.src_cat.identity(x)) = f.dst_cat.identity(f.obj_map(x))
        f.mor_map(opposite_category(f.src_cat).identity(x)) =
            opposite_category(f.dst_cat).identity(f.obj_map(x))
    }
}

/// The same maps preserve reversed composition on opposite categories.
theorem opposite_functor_compose_axiom[O1, M1, O2, M2](f: Functor[O1, M1, O2, M2]) {
    functor_compose_axiom(opposite_category(f.src_cat), opposite_category(f.dst_cat),
        f.obj_map, f.mor_map)
} by {
    forall(p: M1, q: M1) {
        if opposite_category(f.src_cat).src(p) = opposite_category(f.src_cat).dst(q) {
            opposite_category_src(f.src_cat)
            opposite_category_dst(f.src_cat)
            opposite_category(f.src_cat).src = f.src_cat.dst
            opposite_category(f.src_cat).dst = f.src_cat.src
            f.src_cat.dst(p) = f.src_cat.src(q)
            f.src_cat.src(q) = f.src_cat.dst(p)
            functor_compose(f, q, p)
            f.mor_map(f.src_cat.compose(q, p)) =
                f.dst_cat.compose(f.mor_map(q), f.mor_map(p))
            opposite_category_compose(f.src_cat)
            opposite_category(f.src_cat).compose = opposite_compose(f.src_cat)
            opposite_category(f.src_cat).compose(p, q) = opposite_compose(f.src_cat, p, q)
            opposite_compose(f.src_cat, p, q) = f.src_cat.compose(q, p)
            f.mor_map(opposite_category(f.src_cat).compose(p, q)) =
                f.dst_cat.compose(f.mor_map(q), f.mor_map(p))
            opposite_category_compose(f.dst_cat)
            opposite_category(f.dst_cat).compose = opposite_compose(f.dst_cat)
            opposite_category(f.dst_cat).compose(f.mor_map(p), f.mor_map(q)) =
                opposite_compose(f.dst_cat, f.mor_map(p), f.mor_map(q))
            opposite_compose(f.dst_cat, f.mor_map(p), f.mor_map(q)) =
                f.dst_cat.compose(f.mor_map(q), f.mor_map(p))
            f.mor_map(opposite_category(f.src_cat).compose(p, q)) =
                opposite_category(f.dst_cat).compose(f.mor_map(p), f.mor_map(q))
        }
    }
}

/// A functor induces a functor between the opposite categories by reusing its maps.
theorem opposite_is_functor[O1, M1, O2, M2](f: Functor[O1, M1, O2, M2]) {
    is_functor(opposite_category(f.src_cat), opposite_category(f.dst_cat),
        f.obj_map, f.mor_map)
} by {
    opposite_functor_src_axiom(f)
    opposite_functor_dst_axiom(f)
    opposite_functor_identity_axiom(f)
    opposite_functor_compose_axiom(f)
    is_functor(opposite_category(f.src_cat), opposite_category(f.dst_cat),
        f.obj_map, f.mor_map) = (
        functor_src_axiom(opposite_category(f.src_cat), opposite_category(f.dst_cat),
            f.obj_map, f.mor_map)
        and functor_dst_axiom(opposite_category(f.src_cat), opposite_category(f.dst_cat),
            f.obj_map, f.mor_map)
        and functor_identity_axiom(opposite_category(f.src_cat), opposite_category(f.dst_cat),
            f.obj_map, f.mor_map)
        and functor_compose_axiom(opposite_category(f.src_cat), opposite_category(f.dst_cat),
            f.obj_map, f.mor_map))
    functor_src_axiom(opposite_category(f.src_cat), opposite_category(f.dst_cat),
        f.obj_map, f.mor_map)
    functor_dst_axiom(opposite_category(f.src_cat), opposite_category(f.dst_cat),
        f.obj_map, f.mor_map)
    functor_identity_axiom(opposite_category(f.src_cat), opposite_category(f.dst_cat),
        f.obj_map, f.mor_map)
    (functor_src_axiom(opposite_category(f.src_cat), opposite_category(f.dst_cat),
        f.obj_map, f.mor_map)
        and functor_dst_axiom(opposite_category(f.src_cat), opposite_category(f.dst_cat),
            f.obj_map, f.mor_map)
        and functor_identity_axiom(opposite_category(f.src_cat), opposite_category(f.dst_cat),
            f.obj_map, f.mor_map)
        and functor_compose_axiom(opposite_category(f.src_cat), opposite_category(f.dst_cat),
            f.obj_map, f.mor_map))
    is_functor(opposite_category(f.src_cat), opposite_category(f.dst_cat),
        f.obj_map, f.mor_map)
}

/// The opposite functor: same object and morphism maps, with source and target opposite categories.
let opposite_functor[O1, M1, O2, M2](f: Functor[O1, M1, O2, M2]) -> result: Functor[O1, M1, O2, M2] satisfy {
    Functor[O1, M1, O2, M2].new(
        opposite_category(f.src_cat), opposite_category(f.dst_cat), f.obj_map, f.mor_map
    ) = Option.some(result)
} by {
    opposite_is_functor(f)
}

/// The source category of the opposite functor is the opposite source category.
theorem opposite_functor_src_cat[O1, M1, O2, M2](f: Functor[O1, M1, O2, M2]) {
    opposite_functor(f).src_cat = opposite_category(f.src_cat)
} by {
    let op = opposite_functor(f)
    Functor[O1, M1, O2, M2].new(
        opposite_category(f.src_cat), opposite_category(f.dst_cat), f.obj_map, f.mor_map
    ) = Option.some(op)
    functor_new_src_cat(opposite_category(f.src_cat), opposite_category(f.dst_cat),
        f.obj_map, f.mor_map, op)
}

/// The target category of the opposite functor is the opposite target category.
theorem opposite_functor_dst_cat[O1, M1, O2, M2](f: Functor[O1, M1, O2, M2]) {
    opposite_functor(f).dst_cat = opposite_category(f.dst_cat)
} by {
    let op = opposite_functor(f)
    Functor[O1, M1, O2, M2].new(
        opposite_category(f.src_cat), opposite_category(f.dst_cat), f.obj_map, f.mor_map
    ) = Option.some(op)
    functor_new_dst_cat(opposite_category(f.src_cat), opposite_category(f.dst_cat),
        f.obj_map, f.mor_map, op)
}

/// The object map of the opposite functor is unchanged.
theorem opposite_functor_obj_map[O1, M1, O2, M2](f: Functor[O1, M1, O2, M2]) {
    opposite_functor(f).obj_map = f.obj_map
} by {
    let op = opposite_functor(f)
    Functor[O1, M1, O2, M2].new(
        opposite_category(f.src_cat), opposite_category(f.dst_cat), f.obj_map, f.mor_map
    ) = Option.some(op)
    functor_new_obj_map(opposite_category(f.src_cat), opposite_category(f.dst_cat),
        f.obj_map, f.mor_map, op)
}

/// The morphism map of the opposite functor is unchanged.
theorem opposite_functor_mor_map[O1, M1, O2, M2](f: Functor[O1, M1, O2, M2]) {
    opposite_functor(f).mor_map = f.mor_map
} by {
    let op = opposite_functor(f)
    Functor[O1, M1, O2, M2].new(
        opposite_category(f.src_cat), opposite_category(f.dst_cat), f.obj_map, f.mor_map
    ) = Option.some(op)
    functor_new_mor_map(opposite_category(f.src_cat), opposite_category(f.dst_cat),
        f.obj_map, f.mor_map, op)
}

/// Taking the opposite functor twice recovers the original functor.
theorem opposite_functor_involutive[O1, M1, O2, M2](f: Functor[O1, M1, O2, M2]) {
    opposite_functor(opposite_functor(f)) = f
} by {
    let op = opposite_functor(f)
    let opop = opposite_functor(op)
    opposite_functor_src_cat(op)
    opop.src_cat = opposite_category(op.src_cat)
    opposite_functor_src_cat(f)
    op.src_cat = opposite_category(f.src_cat)
    opop.src_cat = opposite_category(opposite_category(f.src_cat))
    opposite_category_involutive(f.src_cat)
    opop.src_cat = f.src_cat
    opposite_functor_dst_cat(op)
    opop.dst_cat = opposite_category(op.dst_cat)
    opposite_functor_dst_cat(f)
    op.dst_cat = opposite_category(f.dst_cat)
    opop.dst_cat = opposite_category(opposite_category(f.dst_cat))
    opposite_category_involutive(f.dst_cat)
    opop.dst_cat = f.dst_cat
    opposite_functor_obj_map(op)
    opop.obj_map = op.obj_map
    opposite_functor_obj_map(f)
    op.obj_map = f.obj_map
    opop.obj_map = f.obj_map
    opposite_functor_mor_map(op)
    opop.mor_map = op.mor_map
    opposite_functor_mor_map(f)
    op.mor_map = f.mor_map
    opop.mor_map = f.mor_map
    functor_ext(opop, f)
}

/// The opposite of an identity functor is the identity functor on the opposite category.
theorem opposite_functor_identity[O, M](c: Category[O, M]) {
    opposite_functor(identity_functor(c)) = identity_functor(opposite_category(c))
} by {
    let lhs = opposite_functor(identity_functor(c))
    let rhs = identity_functor(opposite_category(c))
    opposite_functor_src_cat(identity_functor(c))
    lhs.src_cat = opposite_category(identity_functor(c).src_cat)
    identity_functor_src_cat(c)
    identity_functor(c).src_cat = c
    lhs.src_cat = opposite_category(c)
    identity_functor_src_cat(opposite_category(c))
    rhs.src_cat = opposite_category(c)
    opposite_functor_dst_cat(identity_functor(c))
    lhs.dst_cat = opposite_category(identity_functor(c).dst_cat)
    identity_functor_dst_cat(c)
    identity_functor(c).dst_cat = c
    lhs.dst_cat = opposite_category(c)
    identity_functor_dst_cat(opposite_category(c))
    rhs.dst_cat = opposite_category(c)
    opposite_functor_obj_map(identity_functor(c))
    lhs.obj_map = identity_functor(c).obj_map
    identity_functor_obj_map(c)
    identity_functor(c).obj_map = identity_fn[O]
    lhs.obj_map = identity_fn[O]
    identity_functor_obj_map(opposite_category(c))
    rhs.obj_map = identity_fn[O]
    opposite_functor_mor_map(identity_functor(c))
    lhs.mor_map = identity_functor(c).mor_map
    identity_functor_mor_map(c)
    identity_functor(c).mor_map = identity_fn[M]
    lhs.mor_map = identity_fn[M]
    identity_functor_mor_map(opposite_category(c))
    rhs.mor_map = identity_fn[M]
    functor_ext(lhs, rhs)
}

/// Opposite functors of the source and target of a natural transformation are parallel.
theorem opposite_nat_trans_parallel[O1, M1, O2, M2](
    n: NaturalTransformation[O1, M1, O2, M2]
) {
    functors_parallel(opposite_functor(n.dst_functor), opposite_functor(n.src_functor))
} by {
    let src_op = opposite_functor(n.dst_functor)
    let dst_op = opposite_functor(n.src_functor)
    nat_trans_shared_src_cat(n)
    nat_trans_shared_dst_cat(n)
    opposite_functor_src_cat(n.dst_functor)
    src_op.src_cat = opposite_category(n.dst_functor.src_cat)
    opposite_functor_src_cat(n.src_functor)
    dst_op.src_cat = opposite_category(n.src_functor.src_cat)
    src_op.src_cat = dst_op.src_cat
    opposite_functor_dst_cat(n.dst_functor)
    src_op.dst_cat = opposite_category(n.dst_functor.dst_cat)
    opposite_functor_dst_cat(n.src_functor)
    dst_op.dst_cat = opposite_category(n.src_functor.dst_cat)
    src_op.dst_cat = dst_op.dst_cat
    functors_parallel(src_op, dst_op) = (src_op.src_cat = dst_op.src_cat and src_op.dst_cat = dst_op.dst_cat)
}

/// The components of a natural transformation have reversed endpoints in the opposite target category.
theorem opposite_nat_trans_endpoints[O1, M1, O2, M2](
    n: NaturalTransformation[O1, M1, O2, M2]
) {
    nat_trans_endpoints_axiom(opposite_functor(n.dst_functor), opposite_functor(n.src_functor),
        n.component)
} by {
    let src_op = opposite_functor(n.dst_functor)
    let dst_op = opposite_functor(n.src_functor)
    nat_trans_shared_dst_cat(n)
    opposite_functor_dst_cat(n.dst_functor)
    src_op.dst_cat = opposite_category(n.dst_functor.dst_cat)
    opposite_category_src(n.dst_functor.dst_cat)
    opposite_category(n.dst_functor.dst_cat).src = n.dst_functor.dst_cat.dst
    opposite_category_dst(n.dst_functor.dst_cat)
    opposite_category(n.dst_functor.dst_cat).dst = n.dst_functor.dst_cat.src
    opposite_functor_obj_map(n.dst_functor)
    src_op.obj_map = n.dst_functor.obj_map
    opposite_functor_obj_map(n.src_functor)
    dst_op.obj_map = n.src_functor.obj_map
    forall(x: O1) {
        nat_trans_component_dst(n, x)
        n.src_functor.dst_cat.dst(n.component(x)) = n.dst_functor.obj_map(x)
        n.dst_functor.dst_cat.dst(n.component(x)) = n.dst_functor.obj_map(x)
        src_op.dst_cat.src(n.component(x)) = src_op.obj_map(x)
        nat_trans_component_src(n, x)
        n.src_functor.dst_cat.src(n.component(x)) = n.src_functor.obj_map(x)
        n.dst_functor.dst_cat.src(n.component(x)) = n.src_functor.obj_map(x)
        src_op.dst_cat.dst(n.component(x)) = dst_op.obj_map(x)
        (src_op.dst_cat.src(n.component(x)) = src_op.obj_map(x)
            and src_op.dst_cat.dst(n.component(x)) = dst_op.obj_map(x))
    }
    nat_trans_endpoints_axiom(src_op, dst_op, n.component) = forall(x: O1) {
        src_op.dst_cat.src(n.component(x)) = src_op.obj_map(x)
        and src_op.dst_cat.dst(n.component(x)) = dst_op.obj_map(x)
    }
    nat_trans_endpoints_axiom(src_op, dst_op, n.component)
    nat_trans_endpoints_axiom(opposite_functor(n.dst_functor), opposite_functor(n.src_functor),
        n.component)
}

/// Naturality for a natural transformation is the same equality with both categories opposite.
theorem opposite_nat_trans_naturality[O1, M1, O2, M2](
    n: NaturalTransformation[O1, M1, O2, M2]
) {
    nat_trans_naturality_axiom(opposite_functor(n.dst_functor), opposite_functor(n.src_functor),
        n.component)
} by {
    let src_op = opposite_functor(n.dst_functor)
    let dst_op = opposite_functor(n.src_functor)
    nat_trans_shared_src_cat(n)
    nat_trans_shared_dst_cat(n)
    opposite_functor_src_cat(n.dst_functor)
    src_op.src_cat = opposite_category(n.dst_functor.src_cat)
    opposite_functor_dst_cat(n.dst_functor)
    src_op.dst_cat = opposite_category(n.dst_functor.dst_cat)
    opposite_functor_mor_map(n.dst_functor)
    src_op.mor_map = n.dst_functor.mor_map
    opposite_functor_mor_map(n.src_functor)
    dst_op.mor_map = n.src_functor.mor_map
    opposite_category_src(n.dst_functor.src_cat)
    opposite_category(n.dst_functor.src_cat).src = n.dst_functor.src_cat.dst
    opposite_category_dst(n.dst_functor.src_cat)
    opposite_category(n.dst_functor.src_cat).dst = n.dst_functor.src_cat.src
    opposite_category_compose(n.dst_functor.dst_cat)
    opposite_category(n.dst_functor.dst_cat).compose = opposite_compose(n.dst_functor.dst_cat)
    forall(m: M1) {
        src_op.src_cat.dst(m) = n.dst_functor.src_cat.src(m)
        n.dst_functor.src_cat.src(m) = n.src_functor.src_cat.src(m)
        src_op.src_cat.dst(m) = n.src_functor.src_cat.src(m)
        src_op.src_cat.src(m) = n.dst_functor.src_cat.dst(m)
        n.dst_functor.src_cat.dst(m) = n.src_functor.src_cat.dst(m)
        src_op.src_cat.src(m) = n.src_functor.src_cat.dst(m)
        src_op.mor_map(m) = n.dst_functor.mor_map(m)
        dst_op.mor_map(m) = n.src_functor.mor_map(m)
        src_op.dst_cat.compose(n.component(src_op.src_cat.dst(m)), src_op.mor_map(m)) =
            opposite_compose(n.dst_functor.dst_cat,
                n.component(n.src_functor.src_cat.src(m)), n.dst_functor.mor_map(m))
        opposite_compose(n.dst_functor.dst_cat,
            n.component(n.src_functor.src_cat.src(m)), n.dst_functor.mor_map(m)) =
            n.dst_functor.dst_cat.compose(n.dst_functor.mor_map(m),
                n.component(n.src_functor.src_cat.src(m)))
        n.dst_functor.dst_cat.compose(n.dst_functor.mor_map(m),
            n.component(n.src_functor.src_cat.src(m))) =
            n.src_functor.dst_cat.compose(n.dst_functor.mor_map(m),
                n.component(n.src_functor.src_cat.src(m)))
        src_op.dst_cat.compose(n.component(src_op.src_cat.dst(m)), src_op.mor_map(m)) =
            n.src_functor.dst_cat.compose(n.dst_functor.mor_map(m),
                n.component(n.src_functor.src_cat.src(m)))
        src_op.dst_cat.compose(dst_op.mor_map(m), n.component(src_op.src_cat.src(m))) =
            opposite_compose(n.dst_functor.dst_cat,
                n.src_functor.mor_map(m), n.component(n.src_functor.src_cat.dst(m)))
        opposite_compose(n.dst_functor.dst_cat,
            n.src_functor.mor_map(m), n.component(n.src_functor.src_cat.dst(m))) =
            n.dst_functor.dst_cat.compose(n.component(n.src_functor.src_cat.dst(m)),
                n.src_functor.mor_map(m))
        n.dst_functor.dst_cat.compose(n.component(n.src_functor.src_cat.dst(m)),
            n.src_functor.mor_map(m)) =
            n.src_functor.dst_cat.compose(n.component(n.src_functor.src_cat.dst(m)),
                n.src_functor.mor_map(m))
        nat_trans_naturality(n, m)
        n.src_functor.dst_cat.compose(n.component(n.src_functor.src_cat.dst(m)),
            n.src_functor.mor_map(m)) =
            n.src_functor.dst_cat.compose(n.dst_functor.mor_map(m),
                n.component(n.src_functor.src_cat.src(m)))
        src_op.dst_cat.compose(n.component(src_op.src_cat.dst(m)), src_op.mor_map(m)) =
            src_op.dst_cat.compose(dst_op.mor_map(m), n.component(src_op.src_cat.src(m)))
    }
    nat_trans_naturality_axiom(src_op, dst_op, n.component) = forall(m: M1) {
        src_op.dst_cat.compose(n.component(src_op.src_cat.dst(m)), src_op.mor_map(m)) =
            src_op.dst_cat.compose(dst_op.mor_map(m), n.component(src_op.src_cat.src(m)))
    }
    nat_trans_naturality_axiom(src_op, dst_op, n.component)
    nat_trans_naturality_axiom(opposite_functor(n.dst_functor), opposite_functor(n.src_functor),
        n.component)
}

/// A natural transformation dualizes to a natural transformation between opposite functors, in the reverse direction.
theorem opposite_is_natural_transformation[O1, M1, O2, M2](
    n: NaturalTransformation[O1, M1, O2, M2]
) {
    is_natural_transformation(opposite_functor(n.dst_functor), opposite_functor(n.src_functor),
        n.component)
} by {
    opposite_nat_trans_parallel(n)
    opposite_nat_trans_endpoints(n)
    opposite_nat_trans_naturality(n)
    is_natural_transformation(opposite_functor(n.dst_functor), opposite_functor(n.src_functor),
        n.component) = (
        functors_parallel(opposite_functor(n.dst_functor), opposite_functor(n.src_functor))
        and nat_trans_endpoints_axiom(opposite_functor(n.dst_functor), opposite_functor(n.src_functor),
            n.component)
        and nat_trans_naturality_axiom(opposite_functor(n.dst_functor), opposite_functor(n.src_functor),
            n.component))
    functors_parallel(opposite_functor(n.dst_functor), opposite_functor(n.src_functor))
    nat_trans_endpoints_axiom(opposite_functor(n.dst_functor), opposite_functor(n.src_functor),
        n.component)
    nat_trans_naturality_axiom(opposite_functor(n.dst_functor), opposite_functor(n.src_functor),
        n.component)
    (functors_parallel(opposite_functor(n.dst_functor), opposite_functor(n.src_functor))
        and nat_trans_endpoints_axiom(opposite_functor(n.dst_functor), opposite_functor(n.src_functor),
            n.component)
        and nat_trans_naturality_axiom(opposite_functor(n.dst_functor), opposite_functor(n.src_functor),
            n.component))
    is_natural_transformation(opposite_functor(n.dst_functor), opposite_functor(n.src_functor),
        n.component)
}

/// The dual natural transformation over opposite functors.
let opposite_natural_transformation[O1, M1, O2, M2](n: NaturalTransformation[O1, M1, O2, M2]) -> result: NaturalTransformation[O1, M1, O2, M2] satisfy {
    NaturalTransformation[O1, M1, O2, M2].new(
        opposite_functor(n.dst_functor), opposite_functor(n.src_functor), n.component
    ) = Option.some(result)
} by {
    opposite_is_natural_transformation(n)
}

/// The source functor of the dual natural transformation is the opposite target functor.
theorem opposite_natural_transformation_src_functor[O1, M1, O2, M2](
    n: NaturalTransformation[O1, M1, O2, M2]
) {
    opposite_natural_transformation(n).src_functor = opposite_functor(n.dst_functor)
} by {
    let op = opposite_natural_transformation(n)
    NaturalTransformation[O1, M1, O2, M2].new(
        opposite_functor(n.dst_functor), opposite_functor(n.src_functor), n.component
    ) = Option.some(op)
    nat_trans_new_src_functor(opposite_functor(n.dst_functor), opposite_functor(n.src_functor),
        n.component, op)
}

/// The target functor of the dual natural transformation is the opposite source functor.
theorem opposite_natural_transformation_dst_functor[O1, M1, O2, M2](
    n: NaturalTransformation[O1, M1, O2, M2]
) {
    opposite_natural_transformation(n).dst_functor = opposite_functor(n.src_functor)
} by {
    let op = opposite_natural_transformation(n)
    NaturalTransformation[O1, M1, O2, M2].new(
        opposite_functor(n.dst_functor), opposite_functor(n.src_functor), n.component
    ) = Option.some(op)
    nat_trans_new_dst_functor(opposite_functor(n.dst_functor), opposite_functor(n.src_functor),
        n.component, op)
}

/// The component map of the dual natural transformation is unchanged.
theorem opposite_natural_transformation_component[O1, M1, O2, M2](
    n: NaturalTransformation[O1, M1, O2, M2]
) {
    opposite_natural_transformation(n).component = n.component
} by {
    let op = opposite_natural_transformation(n)
    NaturalTransformation[O1, M1, O2, M2].new(
        opposite_functor(n.dst_functor), opposite_functor(n.src_functor), n.component
    ) = Option.some(op)
    nat_trans_new_component(opposite_functor(n.dst_functor), opposite_functor(n.src_functor),
        n.component, op)
}

/// Dualizing a natural transformation twice recovers the original natural transformation.
theorem opposite_natural_transformation_involutive[O1, M1, O2, M2](
    n: NaturalTransformation[O1, M1, O2, M2]
) {
    opposite_natural_transformation(opposite_natural_transformation(n)) = n
} by {
    let op = opposite_natural_transformation(n)
    let opop = opposite_natural_transformation(op)
    opposite_natural_transformation_src_functor(op)
    opop.src_functor = opposite_functor(op.dst_functor)
    opposite_natural_transformation_dst_functor(n)
    op.dst_functor = opposite_functor(n.src_functor)
    opop.src_functor = opposite_functor(opposite_functor(n.src_functor))
    opposite_functor_involutive(n.src_functor)
    opop.src_functor = n.src_functor
    opposite_natural_transformation_dst_functor(op)
    opop.dst_functor = opposite_functor(op.src_functor)
    opposite_natural_transformation_src_functor(n)
    op.src_functor = opposite_functor(n.dst_functor)
    opop.dst_functor = opposite_functor(opposite_functor(n.dst_functor))
    opposite_functor_involutive(n.dst_functor)
    opop.dst_functor = n.dst_functor
    opposite_natural_transformation_component(op)
    opop.component = op.component
    opposite_natural_transformation_component(n)
    op.component = n.component
    opop.component = n.component
    nat_trans_ext(opop, n)
    opop = n
}
