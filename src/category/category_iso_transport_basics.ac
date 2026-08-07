from category.category import Category, category_compose_assoc, category_id_left, category_id_right
from category.category_iso import Iso, is_iso_pair, iso_is_iso_pair, iso_ext

/// Left composition by an isomorphism is cancellative.
theorem is_iso_pair_cancel_left[O, M](c: Category[O, M], f: M, inv: M, h: M, k: M) {
    is_iso_pair(c, f, inv) and c.src(f) = c.dst(h) and c.src(f) = c.dst(k)
    and c.compose(f, h) = c.compose(f, k) implies h = k
} by {
    if is_iso_pair(c, f, inv) and c.src(f) = c.dst(h) and c.src(f) = c.dst(k)
        and c.compose(f, h) = c.compose(f, k) {
        is_iso_pair(c, f, inv) = (c.src(f) = c.dst(inv)
            and c.src(inv) = c.dst(f)
            and c.compose(inv, f) = c.identity(c.src(f))
            and c.compose(f, inv) = c.identity(c.dst(f)))

        category_compose_assoc(c, inv, f, h)
        c.compose(c.compose(inv, f), h) = c.compose(inv, c.compose(f, h))
        category_id_left(c, h)

        category_compose_assoc(c, inv, f, k)
        c.compose(c.compose(inv, f), k) = c.compose(inv, c.compose(f, k))
        category_id_left(c, k)

        h = k
    }
}

/// Right composition by an isomorphism is cancellative.
theorem is_iso_pair_cancel_right[O, M](c: Category[O, M], f: M, inv: M, h: M, k: M) {
    is_iso_pair(c, f, inv) and c.src(h) = c.dst(f) and c.src(k) = c.dst(f)
    and c.compose(h, f) = c.compose(k, f) implies h = k
} by {
    if is_iso_pair(c, f, inv) and c.src(h) = c.dst(f) and c.src(k) = c.dst(f)
        and c.compose(h, f) = c.compose(k, f) {
        is_iso_pair(c, f, inv) = (c.src(f) = c.dst(inv)
            and c.src(inv) = c.dst(f)
            and c.compose(inv, f) = c.identity(c.src(f))
            and c.compose(f, inv) = c.identity(c.dst(f)))

        category_compose_assoc(c, h, f, inv)
        c.compose(c.compose(h, f), inv) = c.compose(h, c.compose(f, inv))
        category_id_right(c, h)

        category_compose_assoc(c, k, f, inv)
        c.compose(c.compose(k, f), inv) = c.compose(k, c.compose(f, inv))
        category_id_right(c, k)

        h = k
    }
}

/// A morphism has at most one inverse in an isomorphism pair.
theorem is_iso_pair_inverse_unique[O, M](c: Category[O, M], f: M, g: M, h: M) {
    is_iso_pair(c, f, g) and is_iso_pair(c, f, h) implies g = h
} by {
    if is_iso_pair(c, f, g) and is_iso_pair(c, f, h) {
        is_iso_pair(c, f, g) = (c.src(f) = c.dst(g)
            and c.src(g) = c.dst(f)
            and c.compose(g, f) = c.identity(c.src(f))
            and c.compose(f, g) = c.identity(c.dst(f)))
        is_iso_pair(c, f, h) = (c.src(f) = c.dst(h)
            and c.src(h) = c.dst(f)
            and c.compose(h, f) = c.identity(c.src(f))
            and c.compose(f, h) = c.identity(c.dst(f)))
        is_iso_pair_cancel_left(c, f, g, g, h)
        g = h
    }
}

/// Bundled isomorphisms with the same category and forward morphism have the same inverse morphism.
theorem iso_inv_eq_of_cat_hom_eq[O, M](e: Iso[O, M], h: Iso[O, M]) {
    e.cat = h.cat and e.hom = h.hom implies e.inv = h.inv
} by {
    if e.cat = h.cat and e.hom = h.hom {
        iso_is_iso_pair(e)
        iso_is_iso_pair(h)
        is_iso_pair_inverse_unique(e.cat, e.hom, e.inv, h.inv)
        e.inv = h.inv
    }
}

/// Bundled isomorphisms are determined by their category and forward morphism.
theorem iso_eq_of_cat_hom_eq[O, M](e: Iso[O, M], h: Iso[O, M]) {
    e.cat = h.cat and e.hom = h.hom implies e = h
} by {
    if e.cat = h.cat and e.hom = h.hom {
        iso_inv_eq_of_cat_hom_eq(e, h)
        iso_ext(e, h)
        e = h
    }
}
