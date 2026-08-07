/// Isomorphisms in a category.

from category.category import Category, category_identity_src, category_identity_dst,
    category_compose_src, category_compose_dst, category_compose_assoc,
    category_id_left, category_id_right

/// True if `f` and `g` are inverse morphisms in the category `c`.
define is_iso_pair[O, M](c: Category[O, M], f: M, g: M) -> Bool {
    c.src(f) = c.dst(g)
    and c.src(g) = c.dst(f)
    and c.compose(g, f) = c.identity(c.src(f))
    and c.compose(f, g) = c.identity(c.dst(f))
}

/// Swapping an inverse pair gives an inverse pair.
theorem is_iso_pair_swap[O, M](c: Category[O, M], f: M, g: M) {
    is_iso_pair(c, f, g) implies is_iso_pair(c, g, f)
} by {
    if is_iso_pair(c, f, g) {
        is_iso_pair(c, f, g) = (c.src(f) = c.dst(g)
            and c.src(g) = c.dst(f)
            and c.compose(g, f) = c.identity(c.src(f))
            and c.compose(f, g) = c.identity(c.dst(f)))
        c.compose(g, f) = c.identity(c.dst(g))
        is_iso_pair(c, g, f)
    }
}

/// The identity morphism is its own inverse.
theorem is_iso_pair_identity[O, M](c: Category[O, M], x: O) {
    is_iso_pair(c, c.identity(x), c.identity(x))
} by {
    let i = c.identity(x)
    category_identity_src(c, x)
    category_identity_dst(c, x)
    category_id_left(c, i)
    c.compose(i, i) = c.identity(c.src(i))
    c.compose(i, i) = c.identity(c.dst(i))
    is_iso_pair(c, i, i)
}

/// An isomorphism in a category, with explicit inverse data.
structure Iso[O, M] {
    /// The ambient category.
    cat: Category[O, M]

    /// The forward morphism.
    hom: M

    /// The inverse morphism.
    inv: M
} constraint {
    is_iso_pair(cat, hom, inv)
}

/// Construction of an isomorphism remembers the ambient category.
theorem iso_new_cat[O, M](c: Category[O, M], f: M, g: M, e: Iso[O, M]) {
    Iso[O, M].new(c, f, g) = Option.some(e) implies e.cat = c
}

/// Construction of an isomorphism remembers the forward morphism.
theorem iso_new_hom[O, M](c: Category[O, M], f: M, g: M, e: Iso[O, M]) {
    Iso[O, M].new(c, f, g) = Option.some(e) implies e.hom = f
}

/// Construction of an isomorphism remembers the inverse morphism.
theorem iso_new_inv[O, M](c: Category[O, M], f: M, g: M, e: Iso[O, M]) {
    Iso[O, M].new(c, f, g) = Option.some(e) implies e.inv = g
}

/// Every isomorphism is reconstructed from its components.
theorem iso_new_self[O, M](e: Iso[O, M]) {
    Iso[O, M].new(e.cat, e.hom, e.inv) = Option.some(e)
}

/// Equality of options containing isomorphisms is equality of isomorphisms.
theorem iso_some_injective[O, M](e: Iso[O, M], h: Iso[O, M]) {
    Option.some(e) = Option.some(h) implies e = h
}

/// Isomorphisms are equal when their components are equal.
theorem iso_ext[O, M](e: Iso[O, M], h: Iso[O, M]) {
    e.cat = h.cat and e.hom = h.hom and e.inv = h.inv implies e = h
} by {
    if e.cat = h.cat and e.hom = h.hom and e.inv = h.inv {
        Option.some(e) = Option.some(h)
        iso_some_injective(e, h)
    }
}

/// The forward and inverse morphisms of an isomorphism form an inverse pair.
theorem iso_is_iso_pair[O, M](e: Iso[O, M]) {
    is_iso_pair(e.cat, e.hom, e.inv)
} by {
}

/// The source of the forward morphism equals the target of the inverse morphism.
theorem iso_src_hom[O, M](e: Iso[O, M]) {
    e.cat.src(e.hom) = e.cat.dst(e.inv)
} by {
    iso_is_iso_pair(e)
    is_iso_pair(e.cat, e.hom, e.inv) = (e.cat.src(e.hom) = e.cat.dst(e.inv)
        and e.cat.src(e.inv) = e.cat.dst(e.hom)
        and e.cat.compose(e.inv, e.hom) = e.cat.identity(e.cat.src(e.hom))
        and e.cat.compose(e.hom, e.inv) = e.cat.identity(e.cat.dst(e.hom)))
}

/// The source of the inverse morphism equals the target of the forward morphism.
theorem iso_src_inv[O, M](e: Iso[O, M]) {
    e.cat.src(e.inv) = e.cat.dst(e.hom)
} by {
    iso_is_iso_pair(e)
    is_iso_pair(e.cat, e.hom, e.inv) = (e.cat.src(e.hom) = e.cat.dst(e.inv)
        and e.cat.src(e.inv) = e.cat.dst(e.hom)
        and e.cat.compose(e.inv, e.hom) = e.cat.identity(e.cat.src(e.hom))
        and e.cat.compose(e.hom, e.inv) = e.cat.identity(e.cat.dst(e.hom)))
}

/// The composition of the inverse after the forward morphism is the identity at the source.
theorem iso_inv_hom[O, M](e: Iso[O, M]) {
    e.cat.compose(e.inv, e.hom) = e.cat.identity(e.cat.src(e.hom))
} by {
    iso_is_iso_pair(e)
    is_iso_pair(e.cat, e.hom, e.inv) = (e.cat.src(e.hom) = e.cat.dst(e.inv)
        and e.cat.src(e.inv) = e.cat.dst(e.hom)
        and e.cat.compose(e.inv, e.hom) = e.cat.identity(e.cat.src(e.hom))
        and e.cat.compose(e.hom, e.inv) = e.cat.identity(e.cat.dst(e.hom)))
}

/// The composition of the forward after the inverse morphism is the identity at the target.
theorem iso_hom_inv[O, M](e: Iso[O, M]) {
    e.cat.compose(e.hom, e.inv) = e.cat.identity(e.cat.dst(e.hom))
} by {
    iso_is_iso_pair(e)
}

/// The identity morphism on an object as an isomorphism.
let identity_iso[O, M](c: Category[O, M], x: O) -> result: Iso[O, M] satisfy {
    result.cat = c and result.hom = c.identity(x) and result.inv = c.identity(x)
} by {
    is_iso_pair_identity(c, x)
    let e: Iso[O, M] satisfy {
        Iso[O, M].new(c, c.identity(x), c.identity(x)) = Option.some(e)
    }
    iso_new_cat(c, c.identity(x), c.identity(x), e)
    iso_new_hom(c, c.identity(x), c.identity(x), e)
    iso_new_inv(c, c.identity(x), c.identity(x), e)
}

/// The identity isomorphism lives in the given category.
theorem identity_iso_cat[O, M](c: Category[O, M], x: O) {
    identity_iso(c, x).cat = c
}

/// The forward morphism of the identity isomorphism is the identity morphism.
theorem identity_iso_hom[O, M](c: Category[O, M], x: O) {
    identity_iso(c, x).hom = c.identity(x)
}

/// The inverse morphism of the identity isomorphism is the identity morphism.
theorem identity_iso_inv[O, M](c: Category[O, M], x: O) {
    identity_iso(c, x).inv = c.identity(x)
}

/// Swapping the forward and inverse morphisms of an isomorphism.
let inverse_iso[O, M](e: Iso[O, M]) -> result: Iso[O, M] satisfy {
    result.cat = e.cat and result.hom = e.inv and result.inv = e.hom
} by {
    iso_is_iso_pair(e)
    is_iso_pair_swap(e.cat, e.hom, e.inv)
    let h: Iso[O, M] satisfy {
        Iso[O, M].new(e.cat, e.inv, e.hom) = Option.some(h)
    }
    iso_new_cat(e.cat, e.inv, e.hom, h)
    iso_new_hom(e.cat, e.inv, e.hom, h)
    iso_new_inv(e.cat, e.inv, e.hom, h)
}

/// The inverse isomorphism lives in the same category.
theorem inverse_iso_cat[O, M](e: Iso[O, M]) {
    inverse_iso(e).cat = e.cat
}

/// The forward morphism of the inverse isomorphism is the original inverse morphism.
theorem inverse_iso_hom[O, M](e: Iso[O, M]) {
    inverse_iso(e).hom = e.inv
}

/// The inverse morphism of the inverse isomorphism is the original forward morphism.
theorem inverse_iso_inv[O, M](e: Iso[O, M]) {
    inverse_iso(e).inv = e.hom
}

/// Taking the inverse of the inverse recovers the original isomorphism.
theorem inverse_iso_involutive[O, M](e: Iso[O, M]) {
    inverse_iso(inverse_iso(e)) = e
} by {
    let i = inverse_iso(e)
    inverse_iso_cat(e)
    inverse_iso_hom(e)
    inverse_iso_inv(e)
    let j = inverse_iso(i)
    inverse_iso_cat(i)
    inverse_iso_hom(i)
    inverse_iso_inv(i)
    iso_ext(j, e)
}

/// The inverse of the identity isomorphism is the identity isomorphism.
theorem inverse_identity_iso[O, M](c: Category[O, M], x: O) {
    inverse_iso(identity_iso(c, x)) = identity_iso(c, x)
} by {
    let i = identity_iso(c, x)
    identity_iso_cat(c, x)
    identity_iso_hom(c, x)
    identity_iso_inv(c, x)
    let j = inverse_iso(i)
    inverse_iso_cat(i)
    inverse_iso_hom(i)
    inverse_iso_inv(i)
    iso_ext(j, i)
}

/// Helper: combining the inverse identities along the right shows `compose(g1, compose(g2, compose(f2, f1))) = compose(g1, f1)`.
theorem is_iso_pair_compose_back_helper[O, M](c: Category[O, M], f1: M, g1: M, f2: M, g2: M) {
    c.src(g2) = c.dst(f2) and c.compose(g2, f2) = c.identity(c.src(f2))
        and c.src(f2) = c.dst(f1)
        implies c.compose(g2, c.compose(f2, f1)) = f1
} by {
    if c.src(g2) = c.dst(f2) and c.compose(g2, f2) = c.identity(c.src(f2))
        and c.src(f2) = c.dst(f1) {
        category_compose_assoc(c, g2, f2, f1)
        category_id_left(c, f1)
        c.compose(g2, c.compose(f2, f1)) = f1
    }
}

/// Helper: combining the inverse identities along the left shows `compose(f1, compose(g1, g2)) = g2`.
theorem is_iso_pair_compose_fwd_helper[O, M](c: Category[O, M], f1: M, g1: M, f2: M, g2: M) {
    c.src(f1) = c.dst(g1) and c.compose(f1, g1) = c.identity(c.dst(f1))
        and c.dst(f1) = c.dst(g2)
        and c.src(g1) = c.dst(g2)
        implies c.compose(f1, c.compose(g1, g2)) = g2
} by {
    if c.src(f1) = c.dst(g1) and c.compose(f1, g1) = c.identity(c.dst(f1))
        and c.dst(f1) = c.dst(g2)
        and c.src(g1) = c.dst(g2) {
        category_compose_assoc(c, f1, g1, g2)
        category_id_left(c, g2)
        c.compose(f1, c.compose(g1, g2)) = g2
    }
}

/// Composing two inverse pairs gives an inverse pair of the composites.
theorem is_iso_pair_compose[O, M](c: Category[O, M], f1: M, g1: M, f2: M, g2: M) {
    is_iso_pair(c, f1, g1) and is_iso_pair(c, f2, g2)
        and c.src(f2) = c.dst(f1)
        implies is_iso_pair(c, c.compose(f2, f1), c.compose(g1, g2))
} by {
    if is_iso_pair(c, f1, g1) and is_iso_pair(c, f2, g2)
        and c.src(f2) = c.dst(f1) {
        is_iso_pair(c, f1, g1) = (c.src(f1) = c.dst(g1)
            and c.src(g1) = c.dst(f1)
            and c.compose(g1, f1) = c.identity(c.src(f1))
            and c.compose(f1, g1) = c.identity(c.dst(f1)))
        is_iso_pair(c, f2, g2) = (c.src(f2) = c.dst(g2)
            and c.src(g2) = c.dst(f2)
            and c.compose(g2, f2) = c.identity(c.src(f2))
            and c.compose(f2, g2) = c.identity(c.dst(f2)))

        // Composability of (g1, g2)
        c.src(g1) = c.dst(g2)

        let cf = c.compose(f2, f1)
        let cg = c.compose(g1, g2)

        // Endpoint identities for cf
        category_compose_src(c, f2, f1)
        category_compose_dst(c, f2, f1)

        // Endpoint identities for cg
        category_compose_src(c, g1, g2)
        category_compose_dst(c, g1, g2)

        // src(cf) = dst(cg)

        // src(cg) = dst(cf)

        // compose(cg, cf) = identity(src(cf))
        category_compose_assoc(c, g1, g2, c.compose(f2, f1))
        c.compose(c.compose(g1, g2), c.compose(f2, f1)) = c.compose(g1, c.compose(g2, c.compose(f2, f1)))

        is_iso_pair_compose_back_helper(c, f1, g1, f2, g2)
        c.compose(g1, c.compose(g2, c.compose(f2, f1))) = c.compose(g1, f1)
        c.identity(c.src(f1)) = c.identity(c.src(cf))

        // compose(cf, cg) = identity(dst(cf))
        category_compose_assoc(c, f2, f1, c.compose(g1, g2))
        c.compose(c.compose(f2, f1), c.compose(g1, g2)) = c.compose(f2, c.compose(f1, c.compose(g1, g2)))

        is_iso_pair_compose_fwd_helper(c, f1, g1, f2, g2)
        c.compose(f2, c.compose(f1, c.compose(g1, g2))) = c.compose(f2, g2)
        c.identity(c.dst(f2)) = c.identity(c.dst(cf))

        c.src(cf) = c.dst(cg)
        c.src(cg) = c.dst(cf)
        c.compose(cg, cf) = c.identity(c.src(cf))
        c.compose(cf, cg) = c.identity(c.dst(cf))
        is_iso_pair(c, cf, cg)
        is_iso_pair(c, c.compose(f2, f1), c.compose(g1, g2))
    }
}

/// True if two isomorphisms can be composed: same ambient category, with `e2.hom` starting where `e1.hom` ends.
define iso_composable[O, M](e2: Iso[O, M], e1: Iso[O, M]) -> Bool {
    e2.cat = e1.cat and e1.cat.src(e2.hom) = e1.cat.dst(e1.hom)
}

/// The composition of two isomorphisms, packaged as an optional isomorphism.
define iso_compose[O, M](e2: Iso[O, M], e1: Iso[O, M]) -> Option[Iso[O, M]] {
    Iso[O, M].new(
        e1.cat,
        e1.cat.compose(e2.hom, e1.hom),
        e1.cat.compose(e1.inv, e2.inv)
    )
}

/// Two composable isomorphisms compose to some bundled isomorphism.
theorem iso_compose_some[O, M](e2: Iso[O, M], e1: Iso[O, M]) {
    iso_composable(e2, e1) implies exists(h: Iso[O, M]) {
        iso_compose(e2, e1) = Option.some(h)
    }
} by {
    if iso_composable(e2, e1) {
        e2.cat = e1.cat
        e1.cat.src(e2.hom) = e1.cat.dst(e1.hom)
        let c = e1.cat
        iso_is_iso_pair(e1)
        is_iso_pair(c, e1.hom, e1.inv)
        iso_is_iso_pair(e2)
        is_iso_pair(c, e2.hom, e2.inv)
        is_iso_pair_compose(c, e1.hom, e1.inv, e2.hom, e2.inv)
        is_iso_pair(c, c.compose(e2.hom, e1.hom), c.compose(e1.inv, e2.inv))
        let h: Iso[O, M] satisfy {
            Iso[O, M].new(c, c.compose(e2.hom, e1.hom), c.compose(e1.inv, e2.inv)) = Option.some(h)
        }
        iso_compose(e2, e1) = Option.some(h)
    }
}

/// The bundled composition of two isomorphisms lives in the shared ambient category.
theorem iso_compose_cat[O, M](e2: Iso[O, M], e1: Iso[O, M], h: Iso[O, M]) {
    iso_compose(e2, e1) = Option.some(h) implies h.cat = e1.cat
} by {
    if iso_compose(e2, e1) = Option.some(h) {
        Iso[O, M].new(e1.cat, e1.cat.compose(e2.hom, e1.hom), e1.cat.compose(e1.inv, e2.inv)) = Option.some(h)
        iso_new_cat(e1.cat, e1.cat.compose(e2.hom, e1.hom), e1.cat.compose(e1.inv, e2.inv), h)
    }
}

/// The forward morphism of the bundled composition is the composite of the forward morphisms.
theorem iso_compose_hom[O, M](e2: Iso[O, M], e1: Iso[O, M], h: Iso[O, M]) {
    iso_compose(e2, e1) = Option.some(h) implies h.hom = e1.cat.compose(e2.hom, e1.hom)
} by {
    if iso_compose(e2, e1) = Option.some(h) {
        Iso[O, M].new(e1.cat, e1.cat.compose(e2.hom, e1.hom), e1.cat.compose(e1.inv, e2.inv)) = Option.some(h)
        iso_new_hom(e1.cat, e1.cat.compose(e2.hom, e1.hom), e1.cat.compose(e1.inv, e2.inv), h)
    }
}

/// The inverse morphism of the bundled composition is the composite of the inverse morphisms in reverse order.
theorem iso_compose_inv[O, M](e2: Iso[O, M], e1: Iso[O, M], h: Iso[O, M]) {
    iso_compose(e2, e1) = Option.some(h) implies h.inv = e1.cat.compose(e1.inv, e2.inv)
} by {
    if iso_compose(e2, e1) = Option.some(h) {
        Iso[O, M].new(e1.cat, e1.cat.compose(e2.hom, e1.hom), e1.cat.compose(e1.inv, e2.inv)) = Option.some(h)
        iso_new_inv(e1.cat, e1.cat.compose(e2.hom, e1.hom), e1.cat.compose(e1.inv, e2.inv), h)
    }
}

/// Composing the identity isomorphism on the target on the left with an isomorphism recovers the original isomorphism.
theorem iso_compose_identity_left[O, M](e: Iso[O, M]) {
    iso_compose(identity_iso(e.cat, e.cat.dst(e.hom)), e) = Option.some(e)
} by {
    let c = e.cat
    let x = c.dst(e.hom)
    let i = identity_iso(c, x)
    identity_iso_cat(c, x)
    i.cat = c
    identity_iso_hom(c, x)
    i.hom = c.identity(x)
    identity_iso_inv(c, x)
    i.inv = c.identity(x)
    category_identity_src(c, x)
    c.src(c.identity(x)) = x
    c.src(i.hom) = x
    c.src(i.hom) = c.dst(e.hom)
    iso_composable(i, e) = (i.cat = e.cat and e.cat.src(i.hom) = e.cat.dst(e.hom))
    iso_composable(i, e)
    iso_compose_some(i, e)
    let h: Iso[O, M] satisfy {
        iso_compose(i, e) = Option.some(h)
    }
    iso_compose_cat(i, e, h)
    iso_compose_hom(i, e, h)
    category_id_left(c, e.hom)
    iso_compose_inv(i, e, h)
    iso_src_inv(e)
    category_id_right(c, e.inv)
    h.inv = e.inv
    iso_ext(h, e)
}

/// Composing an isomorphism on the left with the identity isomorphism on its source recovers the original isomorphism.
theorem iso_compose_identity_right[O, M](e: Iso[O, M]) {
    iso_compose(e, identity_iso(e.cat, e.cat.src(e.hom))) = Option.some(e)
} by {
    let c = e.cat
    let x = c.src(e.hom)
    let i = identity_iso(c, x)
    identity_iso_cat(c, x)
    i.cat = c
    identity_iso_hom(c, x)
    i.hom = c.identity(x)
    identity_iso_inv(c, x)
    i.inv = c.identity(x)
    category_identity_dst(c, x)
    c.dst(c.identity(x)) = x
    c.dst(i.hom) = x
    c.dst(i.hom) = c.src(e.hom)
    iso_src_hom(e)
    c.src(e.hom) = c.dst(e.inv)
    iso_composable(e, i) = (e.cat = i.cat and i.cat.src(e.hom) = i.cat.dst(i.hom))
    c.src(e.hom) = c.dst(i.hom)
    iso_composable(e, i)
    iso_compose_some(e, i)
    let h: Iso[O, M] satisfy {
        iso_compose(e, i) = Option.some(h)
    }
    iso_compose_cat(e, i, h)
    iso_compose_hom(e, i, h)
    category_id_right(c, e.hom)
    h.hom = e.hom
    iso_compose_inv(e, i, h)
    category_id_left(c, e.inv)
    h.inv = e.inv
    iso_ext(h, e)
}

/// The inverse of a composite isomorphism is the composite of the inverses in reverse order.
theorem inverse_iso_compose[O, M](e2: Iso[O, M], e1: Iso[O, M], h: Iso[O, M]) {
    iso_composable(e2, e1) and iso_compose(e2, e1) = Option.some(h) implies
        iso_compose(inverse_iso(e1), inverse_iso(e2)) = Option.some(inverse_iso(h))
} by {
    if iso_composable(e2, e1) and iso_compose(e2, e1) = Option.some(h) {
        let c = e1.cat
        iso_composable(e2, e1) = (e2.cat = e1.cat and e1.cat.src(e2.hom) = e1.cat.dst(e1.hom))
        e2.cat = e1.cat
        c.src(e2.hom) = c.dst(e1.hom)
        iso_compose_cat(e2, e1, h)
        h.cat = c
        iso_compose_hom(e2, e1, h)
        h.hom = c.compose(e2.hom, e1.hom)
        iso_compose_inv(e2, e1, h)
        h.inv = c.compose(e1.inv, e2.inv)

        let a = inverse_iso(e1)
        inverse_iso_cat(e1)
        a.cat = e1.cat
        a.cat = c
        inverse_iso_hom(e1)
        a.hom = e1.inv
        inverse_iso_inv(e1)
        a.inv = e1.hom

        let b = inverse_iso(e2)
        inverse_iso_cat(e2)
        b.cat = e2.cat
        b.cat = c
        inverse_iso_hom(e2)
        b.hom = e2.inv
        inverse_iso_inv(e2)
        b.inv = e2.hom

        // Composability of (a, b)
        a.cat = b.cat
        iso_src_inv(e1)
        c.src(e1.inv) = c.dst(e1.hom)
        c.src(a.hom) = c.dst(e1.hom)
        iso_src_hom(e2)
        c.src(e2.hom) = c.dst(e2.inv)
        c.dst(e1.hom) = c.src(e2.hom)
        c.dst(e1.hom) = c.dst(e2.inv)
        c.src(a.hom) = c.dst(e2.inv)
        c.dst(b.hom) = c.dst(e2.inv)
        c.src(a.hom) = c.dst(b.hom)
        b.cat.src(a.hom) = b.cat.dst(b.hom)
        iso_composable(a, b) = (a.cat = b.cat and b.cat.src(a.hom) = b.cat.dst(b.hom))
        iso_composable(a, b)
        iso_compose_some(a, b)
        let k: Iso[O, M] satisfy {
            iso_compose(a, b) = Option.some(k)
        }
        iso_compose_cat(a, b, k)
        k.cat = b.cat
        k.cat = c
        iso_compose_hom(a, b, k)
        k.hom = b.cat.compose(a.hom, b.hom)
        k.hom = c.compose(e1.inv, e2.inv)
        iso_compose_inv(a, b, k)
        k.inv = b.cat.compose(b.inv, a.inv)
        k.inv = c.compose(e2.hom, e1.hom)

        let hi = inverse_iso(h)
        inverse_iso_cat(h)
        hi.cat = h.cat
        hi.cat = c
        inverse_iso_hom(h)
        hi.hom = h.inv
        hi.hom = c.compose(e1.inv, e2.inv)
        inverse_iso_inv(h)
        hi.inv = h.hom
        hi.inv = c.compose(e2.hom, e1.hom)

        iso_ext(k, hi)
        iso_compose(a, b) = Option.some(hi)
    }
}

/// The source of the composite isomorphism's forward morphism equals the source of the inner forward morphism.
theorem iso_compose_src_hom[O, M](e2: Iso[O, M], e1: Iso[O, M], h: Iso[O, M]) {
    iso_composable(e2, e1) and iso_compose(e2, e1) = Option.some(h) implies
        e1.cat.src(h.hom) = e1.cat.src(e1.hom)
} by {
    if iso_composable(e2, e1) and iso_compose(e2, e1) = Option.some(h) {
        iso_compose_hom(e2, e1, h)
        category_compose_src(e1.cat, e2.hom, e1.hom)
        e1.cat.src(e1.cat.compose(e2.hom, e1.hom)) = e1.cat.src(e1.hom)
    }
}

/// The target of the composite isomorphism's forward morphism equals the target of the outer forward morphism.
theorem iso_compose_dst_hom[O, M](e2: Iso[O, M], e1: Iso[O, M], h: Iso[O, M]) {
    iso_composable(e2, e1) and iso_compose(e2, e1) = Option.some(h) implies
        e1.cat.dst(h.hom) = e1.cat.dst(e2.hom)
} by {
    if iso_composable(e2, e1) and iso_compose(e2, e1) = Option.some(h) {
        iso_compose_hom(e2, e1, h)
        category_compose_dst(e1.cat, e2.hom, e1.hom)
        e1.cat.dst(e1.cat.compose(e2.hom, e1.hom)) = e1.cat.dst(e2.hom)
    }
}

/// The source of the composite isomorphism's inverse morphism equals the source of the outer inverse morphism.
theorem iso_compose_src_inv[O, M](e2: Iso[O, M], e1: Iso[O, M], h: Iso[O, M]) {
    iso_composable(e2, e1) and iso_compose(e2, e1) = Option.some(h) implies
        e1.cat.src(h.inv) = e1.cat.src(e2.inv)
} by {
    if iso_composable(e2, e1) and iso_compose(e2, e1) = Option.some(h) {
        iso_composable(e2, e1) = (e2.cat = e1.cat and e1.cat.src(e2.hom) = e1.cat.dst(e1.hom))
        iso_src_inv(e1)
        iso_src_hom(e2)
        iso_compose_inv(e2, e1, h)
        category_compose_src(e1.cat, e1.inv, e2.inv)
        e1.cat.src(e1.cat.compose(e1.inv, e2.inv)) = e1.cat.src(e2.inv)
    }
}

/// The target of the composite isomorphism's inverse morphism equals the target of the inner inverse morphism.
theorem iso_compose_dst_inv[O, M](e2: Iso[O, M], e1: Iso[O, M], h: Iso[O, M]) {
    iso_composable(e2, e1) and iso_compose(e2, e1) = Option.some(h) implies
        e1.cat.dst(h.inv) = e1.cat.dst(e1.inv)
} by {
    if iso_composable(e2, e1) and iso_compose(e2, e1) = Option.some(h) {
        iso_composable(e2, e1) = (e2.cat = e1.cat and e1.cat.src(e2.hom) = e1.cat.dst(e1.hom))
        iso_src_inv(e1)
        iso_src_hom(e2)
        iso_compose_inv(e2, e1, h)
        category_compose_dst(e1.cat, e1.inv, e2.inv)
        e1.cat.dst(e1.cat.compose(e1.inv, e2.inv)) = e1.cat.dst(e1.inv)
    }
}

/// If `e3` follows `e2` and `e2` follows `e1`, then `e3` follows the composite `e2 ∘ e1`.
theorem iso_compose_composable_left[O, M](
    e3: Iso[O, M], e2: Iso[O, M], e1: Iso[O, M], e21: Iso[O, M]
) {
    iso_composable(e2, e1) and iso_composable(e3, e2)
    and iso_compose(e2, e1) = Option.some(e21)
    implies iso_composable(e3, e21)
} by {
    if iso_composable(e2, e1) and iso_composable(e3, e2)
    and iso_compose(e2, e1) = Option.some(e21) {
        iso_composable(e2, e1) = (e2.cat = e1.cat and e1.cat.src(e2.hom) = e1.cat.dst(e1.hom))
        iso_composable(e3, e2) = (e3.cat = e2.cat and e2.cat.src(e3.hom) = e2.cat.dst(e2.hom))
        e2.cat = e1.cat
        e3.cat = e2.cat
        e3.cat = e1.cat
        iso_compose_cat(e2, e1, e21)
        e21.cat = e1.cat
        e3.cat = e21.cat
        e2.cat.src(e3.hom) = e2.cat.dst(e2.hom)
        e1.cat.src(e3.hom) = e1.cat.dst(e2.hom)
        iso_compose_dst_hom(e2, e1, e21)
        e1.cat.dst(e21.hom) = e1.cat.dst(e2.hom)
        e1.cat.src(e3.hom) = e1.cat.dst(e21.hom)
        e21.cat.src(e3.hom) = e21.cat.dst(e21.hom)
        iso_composable(e3, e21) = (e3.cat = e21.cat and e21.cat.src(e3.hom) = e21.cat.dst(e21.hom))
        iso_composable(e3, e21)
    }
}

/// If `e3` follows `e2` and `e2` follows `e1`, then the composite `e3 ∘ e2` follows `e1`.
theorem iso_compose_composable_right[O, M](
    e3: Iso[O, M], e2: Iso[O, M], e1: Iso[O, M], e32: Iso[O, M]
) {
    iso_composable(e2, e1) and iso_composable(e3, e2)
    and iso_compose(e3, e2) = Option.some(e32)
    implies iso_composable(e32, e1)
} by {
    if iso_composable(e2, e1) and iso_composable(e3, e2)
    and iso_compose(e3, e2) = Option.some(e32) {
        iso_composable(e2, e1) = (e2.cat = e1.cat and e1.cat.src(e2.hom) = e1.cat.dst(e1.hom))
        iso_composable(e3, e2) = (e3.cat = e2.cat and e2.cat.src(e3.hom) = e2.cat.dst(e2.hom))
        e2.cat = e1.cat
        e3.cat = e2.cat
        iso_compose_cat(e3, e2, e32)
        e32.cat = e2.cat
        e32.cat = e1.cat
        e1.cat.src(e2.hom) = e1.cat.dst(e1.hom)
        iso_compose_src_hom(e3, e2, e32)
        e2.cat.src(e32.hom) = e2.cat.src(e2.hom)
        e1.cat.src(e32.hom) = e1.cat.src(e2.hom)
        e1.cat.src(e32.hom) = e1.cat.dst(e1.hom)
        iso_composable(e32, e1) = (e32.cat = e1.cat and e1.cat.src(e32.hom) = e1.cat.dst(e1.hom))
        iso_composable(e32, e1)
    }
}

/// Bundled isomorphism composition is associative for explicitly supplied composite witnesses.
theorem iso_compose_assoc_eq[O, M](
    e3: Iso[O, M], e2: Iso[O, M], e1: Iso[O, M],
    e21: Iso[O, M], e32: Iso[O, M], left: Iso[O, M], right: Iso[O, M]
) {
    iso_composable(e2, e1) and iso_composable(e3, e2)
    and iso_compose(e2, e1) = Option.some(e21)
    and iso_compose(e3, e2) = Option.some(e32)
    and iso_compose(e3, e21) = Option.some(left)
    and iso_compose(e32, e1) = Option.some(right)
    implies left = right
} by {
    if iso_composable(e2, e1) and iso_composable(e3, e2)
    and iso_compose(e2, e1) = Option.some(e21)
    and iso_compose(e3, e2) = Option.some(e32)
    and iso_compose(e3, e21) = Option.some(left)
    and iso_compose(e32, e1) = Option.some(right) {
        let c = e1.cat
        iso_composable(e2, e1) = (e2.cat = e1.cat and e1.cat.src(e2.hom) = e1.cat.dst(e1.hom))
        iso_composable(e3, e2) = (e3.cat = e2.cat and e2.cat.src(e3.hom) = e2.cat.dst(e2.hom))
        e2.cat = c
        e3.cat = e2.cat
        e3.cat = c
        c.src(e2.hom) = c.dst(e1.hom)
        e2.cat.src(e3.hom) = e2.cat.dst(e2.hom)
        e2.cat.src = c.src
        e2.cat.dst = c.dst
        c.src(e3.hom) = e2.cat.src(e3.hom)
        c.dst(e2.hom) = e2.cat.dst(e2.hom)
        c.src(e3.hom) = c.dst(e2.hom)

        iso_compose_cat(e2, e1, e21)
        e21.cat = c
        iso_compose_hom(e2, e1, e21)
        e21.hom = c.compose(e2.hom, e1.hom)
        iso_compose_inv(e2, e1, e21)
        e21.inv = c.compose(e1.inv, e2.inv)

        iso_compose_cat(e3, e2, e32)
        e32.cat = e2.cat
        e32.cat = c
        iso_compose_hom(e3, e2, e32)
        e32.hom = e2.cat.compose(e3.hom, e2.hom)
        e32.hom = c.compose(e3.hom, e2.hom)
        iso_compose_inv(e3, e2, e32)
        e32.inv = e2.cat.compose(e2.inv, e3.inv)
        e32.inv = c.compose(e2.inv, e3.inv)

        iso_compose_cat(e3, e21, left)
        left.cat = e21.cat
        left.cat = c
        iso_compose_hom(e3, e21, left)
        left.hom = e21.cat.compose(e3.hom, e21.hom)
        left.hom = c.compose(e3.hom, c.compose(e2.hom, e1.hom))
        iso_compose_inv(e3, e21, left)
        left.inv = e21.cat.compose(e21.inv, e3.inv)
        left.inv = c.compose(c.compose(e1.inv, e2.inv), e3.inv)

        iso_compose_cat(e32, e1, right)
        right.cat = e1.cat
        right.cat = c
        iso_compose_hom(e32, e1, right)
        right.hom = c.compose(e32.hom, e1.hom)
        right.hom = c.compose(c.compose(e3.hom, e2.hom), e1.hom)
        iso_compose_inv(e32, e1, right)
        right.inv = c.compose(e1.inv, e32.inv)
        right.inv = c.compose(e1.inv, c.compose(e2.inv, e3.inv))

        category_compose_assoc(c, e3.hom, e2.hom, e1.hom)
        c.compose(c.compose(e3.hom, e2.hom), e1.hom) = c.compose(e3.hom, c.compose(e2.hom, e1.hom))
        left.hom = right.hom

        iso_src_inv(e1)
        c.src(e1.inv) = c.dst(e1.hom)
        iso_src_hom(e2)
        e2.cat.src(e2.hom) = e2.cat.dst(e2.inv)
        c.src(e2.hom) = c.dst(e2.inv)
        c.src(e1.inv) = c.dst(e2.inv)
        iso_src_inv(e2)
        e2.cat.src(e2.inv) = e2.cat.dst(e2.hom)
        c.src(e2.inv) = c.dst(e2.hom)
        iso_src_hom(e3)
        e3.cat.src(e3.hom) = e3.cat.dst(e3.inv)
        c.src(e3.hom) = c.dst(e3.inv)
        c.src(e2.inv) = c.dst(e3.inv)
        category_compose_assoc(c, e1.inv, e2.inv, e3.inv)
        c.compose(c.compose(e1.inv, e2.inv), e3.inv) = c.compose(e1.inv, c.compose(e2.inv, e3.inv))
        left.inv = right.inv

        iso_ext(left, right)
        left = right
    }
}

/// A composable triple of isomorphisms has equal left- and right-associated bundled composites.
theorem iso_compose_assoc_some[O, M](e3: Iso[O, M], e2: Iso[O, M], e1: Iso[O, M]) {
    iso_composable(e2, e1) and iso_composable(e3, e2) implies
    exists(left: Iso[O, M], right: Iso[O, M]) {
        exists(e21: Iso[O, M], e32: Iso[O, M]) {
            iso_compose(e2, e1) = Option.some(e21)
            and iso_compose(e3, e2) = Option.some(e32)
            and iso_compose(e3, e21) = Option.some(left)
            and iso_compose(e32, e1) = Option.some(right)
            and left = right
        }
    }
} by {
    if iso_composable(e2, e1) and iso_composable(e3, e2) {
        iso_compose_some(e2, e1)
        let e21: Iso[O, M] satisfy {
            iso_compose(e2, e1) = Option.some(e21)
        }
        iso_compose_some(e3, e2)
        let e32: Iso[O, M] satisfy {
            iso_compose(e3, e2) = Option.some(e32)
        }
        iso_compose_composable_left(e3, e2, e1, e21)
        iso_composable(e3, e21)
        iso_compose_some(e3, e21)
        let left: Iso[O, M] satisfy {
            iso_compose(e3, e21) = Option.some(left)
        }
        iso_compose_composable_right(e3, e2, e1, e32)
        iso_composable(e32, e1)
        iso_compose_some(e32, e1)
        let right: Iso[O, M] satisfy {
            iso_compose(e32, e1) = Option.some(right)
        }
        iso_compose_assoc_eq(e3, e2, e1, e21, e32, left, right)
        left = right
        exists(l: Iso[O, M], r: Iso[O, M]) {
            exists(a: Iso[O, M], b: Iso[O, M]) {
                iso_compose(e2, e1) = Option.some(a)
                and iso_compose(e3, e2) = Option.some(b)
                and iso_compose(e3, a) = Option.some(l)
                and iso_compose(b, e1) = Option.some(r)
                and l = r
            }
        }
    }
}
