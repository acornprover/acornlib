/// Functors between categories.

from category.category import Category, category_identity_src, category_identity_dst,
    category_compose_src, category_compose_dst, category_compose_assoc,
    category_id_left, category_id_right
from data.basic.functions import compose, identity_fn, compose_identity_left, compose_identity_right,
    function_extensionality

/// True if `mor_map` sends sources in `c` to sources in `d`, after `obj_map`.
define functor_src_axiom[O1, M1, O2, M2](
    c: Category[O1, M1], d: Category[O2, M2],
    obj_map: O1 -> O2, mor_map: M1 -> M2
) -> Bool {
    forall(f: M1) {
        d.src(mor_map(f)) = obj_map(c.src(f))
    }
}

/// True if `mor_map` sends targets in `c` to targets in `d`, after `obj_map`.
define functor_dst_axiom[O1, M1, O2, M2](
    c: Category[O1, M1], d: Category[O2, M2],
    obj_map: O1 -> O2, mor_map: M1 -> M2
) -> Bool {
    forall(f: M1) {
        d.dst(mor_map(f)) = obj_map(c.dst(f))
    }
}

/// True if `mor_map` sends identities in `c` to identities in `d`.
define functor_identity_axiom[O1, M1, O2, M2](
    c: Category[O1, M1], d: Category[O2, M2],
    obj_map: O1 -> O2, mor_map: M1 -> M2
) -> Bool {
    forall(x: O1) {
        mor_map(c.identity(x)) = d.identity(obj_map(x))
    }
}

/// True if `mor_map` sends compositions in `c` to compositions in `d`.
define functor_compose_axiom[O1, M1, O2, M2](
    c: Category[O1, M1], d: Category[O2, M2],
    obj_map: O1 -> O2, mor_map: M1 -> M2
) -> Bool {
    forall(f: M1, g: M1) {
        c.src(f) = c.dst(g) implies mor_map(c.compose(f, g)) = d.compose(mor_map(f), mor_map(g))
    }
}

/// True if `obj_map` and `mor_map` form a functor from `c` to `d`.
define is_functor[O1, M1, O2, M2](
    c: Category[O1, M1], d: Category[O2, M2],
    obj_map: O1 -> O2, mor_map: M1 -> M2
) -> Bool {
    functor_src_axiom(c, d, obj_map, mor_map)
    and functor_dst_axiom(c, d, obj_map, mor_map)
    and functor_identity_axiom(c, d, obj_map, mor_map)
    and functor_compose_axiom(c, d, obj_map, mor_map)
}

/// A functor between two categories.
structure Functor[O1, M1, O2, M2] {
    /// The source category.
    src_cat: Category[O1, M1]

    /// The target category.
    dst_cat: Category[O2, M2]

    /// The action on objects.
    obj_map: O1 -> O2

    /// The action on morphisms.
    mor_map: M1 -> M2
} constraint {
    is_functor(src_cat, dst_cat, obj_map, mor_map)
}

/// Functor extensionality from equality of categories and maps.
theorem functor_ext[O1, M1, O2, M2](
    f: Functor[O1, M1, O2, M2],
    g: Functor[O1, M1, O2, M2]
) {
    f.src_cat = g.src_cat and f.dst_cat = g.dst_cat and
    f.obj_map = g.obj_map and f.mor_map = g.mor_map implies f = g
}

/// Construction of a functor remembers its source category.
theorem functor_new_src_cat[O1, M1, O2, M2](
    c: Category[O1, M1], d: Category[O2, M2],
    obj_map: O1 -> O2, mor_map: M1 -> M2,
    f: Functor[O1, M1, O2, M2]
) {
    Functor[O1, M1, O2, M2].new(c, d, obj_map, mor_map) = Option.some(f) implies f.src_cat = c
}

/// Construction of a functor remembers its target category.
theorem functor_new_dst_cat[O1, M1, O2, M2](
    c: Category[O1, M1], d: Category[O2, M2],
    obj_map: O1 -> O2, mor_map: M1 -> M2,
    f: Functor[O1, M1, O2, M2]
) {
    Functor[O1, M1, O2, M2].new(c, d, obj_map, mor_map) = Option.some(f) implies f.dst_cat = d
}

/// Construction of a functor remembers its action on objects.
theorem functor_new_obj_map[O1, M1, O2, M2](
    c: Category[O1, M1], d: Category[O2, M2],
    obj_map: O1 -> O2, mor_map: M1 -> M2,
    f: Functor[O1, M1, O2, M2]
) {
    Functor[O1, M1, O2, M2].new(c, d, obj_map, mor_map) = Option.some(f) implies f.obj_map = obj_map
}

/// Construction of a functor remembers its action on morphisms.
theorem functor_new_mor_map[O1, M1, O2, M2](
    c: Category[O1, M1], d: Category[O2, M2],
    obj_map: O1 -> O2, mor_map: M1 -> M2,
    f: Functor[O1, M1, O2, M2]
) {
    Functor[O1, M1, O2, M2].new(c, d, obj_map, mor_map) = Option.some(f) implies f.mor_map = mor_map
}

/// A functor's data forms a functor from its source to its target.
theorem functor_is_functor[O1, M1, O2, M2](f: Functor[O1, M1, O2, M2]) {
    is_functor(f.src_cat, f.dst_cat, f.obj_map, f.mor_map)
} by {
}

/// A functor preserves the source of every morphism.
theorem functor_src[O1, M1, O2, M2](f: Functor[O1, M1, O2, M2], m: M1) {
    f.dst_cat.src(f.mor_map(m)) = f.obj_map(f.src_cat.src(m))
} by {
    functor_is_functor(f)
    is_functor(f.src_cat, f.dst_cat, f.obj_map, f.mor_map) =
        (functor_src_axiom(f.src_cat, f.dst_cat, f.obj_map, f.mor_map)
        and functor_dst_axiom(f.src_cat, f.dst_cat, f.obj_map, f.mor_map)
        and functor_identity_axiom(f.src_cat, f.dst_cat, f.obj_map, f.mor_map)
        and functor_compose_axiom(f.src_cat, f.dst_cat, f.obj_map, f.mor_map))
    functor_src_axiom(f.src_cat, f.dst_cat, f.obj_map, f.mor_map) = forall(g: M1) {
        f.dst_cat.src(f.mor_map(g)) = f.obj_map(f.src_cat.src(g))
    }
}

/// A functor preserves the target of every morphism.
theorem functor_dst[O1, M1, O2, M2](f: Functor[O1, M1, O2, M2], m: M1) {
    f.dst_cat.dst(f.mor_map(m)) = f.obj_map(f.src_cat.dst(m))
} by {
    functor_is_functor(f)
    is_functor(f.src_cat, f.dst_cat, f.obj_map, f.mor_map) =
        (functor_src_axiom(f.src_cat, f.dst_cat, f.obj_map, f.mor_map)
        and functor_dst_axiom(f.src_cat, f.dst_cat, f.obj_map, f.mor_map)
        and functor_identity_axiom(f.src_cat, f.dst_cat, f.obj_map, f.mor_map)
        and functor_compose_axiom(f.src_cat, f.dst_cat, f.obj_map, f.mor_map))
    functor_dst_axiom(f.src_cat, f.dst_cat, f.obj_map, f.mor_map) = forall(g: M1) {
        f.dst_cat.dst(f.mor_map(g)) = f.obj_map(f.src_cat.dst(g))
    }
}

/// A functor sends identities to identities.
theorem functor_identity[O1, M1, O2, M2](f: Functor[O1, M1, O2, M2], x: O1) {
    f.mor_map(f.src_cat.identity(x)) = f.dst_cat.identity(f.obj_map(x))
} by {
    functor_is_functor(f)
    is_functor(f.src_cat, f.dst_cat, f.obj_map, f.mor_map) =
        (functor_src_axiom(f.src_cat, f.dst_cat, f.obj_map, f.mor_map)
        and functor_dst_axiom(f.src_cat, f.dst_cat, f.obj_map, f.mor_map)
        and functor_identity_axiom(f.src_cat, f.dst_cat, f.obj_map, f.mor_map)
        and functor_compose_axiom(f.src_cat, f.dst_cat, f.obj_map, f.mor_map))
    functor_identity_axiom(f.src_cat, f.dst_cat, f.obj_map, f.mor_map) = forall(y: O1) {
        f.mor_map(f.src_cat.identity(y)) = f.dst_cat.identity(f.obj_map(y))
    }
}

/// A functor preserves composition of composable morphisms.
theorem functor_compose[O1, M1, O2, M2](f: Functor[O1, M1, O2, M2], g: M1, h: M1) {
    f.src_cat.src(g) = f.src_cat.dst(h) implies
        f.mor_map(f.src_cat.compose(g, h)) = f.dst_cat.compose(f.mor_map(g), f.mor_map(h))
} by {
    functor_is_functor(f)
    functor_compose_axiom(f.src_cat, f.dst_cat, f.obj_map, f.mor_map)
    functor_compose_axiom(f.src_cat, f.dst_cat, f.obj_map, f.mor_map) = forall(p: M1, q: M1) {
        f.src_cat.src(p) = f.src_cat.dst(q) implies
            f.mor_map(f.src_cat.compose(p, q)) = f.dst_cat.compose(f.mor_map(p), f.mor_map(q))
    }
}

/// The identity maps form a functor from a category to itself.
theorem identity_is_functor[O, M](c: Category[O, M]) {
    is_functor(c, c, identity_fn[O], identity_fn[M])
} by {
    forall(g: M) {
        c.src(identity_fn[M](g)) = identity_fn[O](c.src(g))
    }
    functor_src_axiom(c, c, identity_fn[O], identity_fn[M])
    forall(g: M) {
        c.dst(identity_fn[M](g)) = identity_fn[O](c.dst(g))
    }
    functor_dst_axiom(c, c, identity_fn[O], identity_fn[M])
    forall(x: O) {
        identity_fn[M](c.identity(x)) = c.identity(identity_fn[O](x))
    }
    functor_identity_axiom(c, c, identity_fn[O], identity_fn[M])
    forall(p: M, q: M) {
        if c.src(p) = c.dst(q) {
            identity_fn[M](c.compose(p, q)) = c.compose(identity_fn[M](p), identity_fn[M](q))
        }
    }
    functor_compose_axiom(c, c, identity_fn[O], identity_fn[M])
}

/// The identity functor on a category.
let identity_functor[O, M](c: Category[O, M]) -> result: Functor[O, M, O, M] satisfy {
    Functor[O, M, O, M].new(c, c, identity_fn[O], identity_fn[M]) = Option.some(result)
} by {
    identity_is_functor(c)
}

/// The identity functor has the original category as its source.
theorem identity_functor_src_cat[O, M](c: Category[O, M]) {
    identity_functor(c).src_cat = c
} by {
    let f = identity_functor(c)
    (Functor[O, M, O, M].new(c, c, identity_fn[O], identity_fn[M]) = Option.some(f))
    functor_new_src_cat(c, c, identity_fn[O], identity_fn[M], f)
}

/// The identity functor has the original category as its target.
theorem identity_functor_dst_cat[O, M](c: Category[O, M]) {
    identity_functor(c).dst_cat = c
} by {
    let f = identity_functor(c)
    (Functor[O, M, O, M].new(c, c, identity_fn[O], identity_fn[M]) = Option.some(f))
    functor_new_dst_cat(c, c, identity_fn[O], identity_fn[M], f)
}

/// The identity functor has the identity function as its action on objects.
theorem identity_functor_obj_map[O, M](c: Category[O, M]) {
    identity_functor(c).obj_map = identity_fn[O]
} by {
    let f = identity_functor(c)
    (Functor[O, M, O, M].new(c, c, identity_fn[O], identity_fn[M]) = Option.some(f))
    functor_new_obj_map(c, c, identity_fn[O], identity_fn[M], f)
}

/// The identity functor has the identity function as its action on morphisms.
theorem identity_functor_mor_map[O, M](c: Category[O, M]) {
    identity_functor(c).mor_map = identity_fn[M]
} by {
    let f = identity_functor(c)
    (Functor[O, M, O, M].new(c, c, identity_fn[O], identity_fn[M]) = Option.some(f))
    functor_new_mor_map(c, c, identity_fn[O], identity_fn[M], f)
}

/// The composition of two functors satisfies the source axiom.
theorem compose_functor_src_axiom[O1, M1, O2, M2, O3, M3](
    f: Functor[O2, M2, O3, M3], g: Functor[O1, M1, O2, M2]
) {
    f.src_cat = g.dst_cat implies
        functor_src_axiom(g.src_cat, f.dst_cat, compose(f.obj_map, g.obj_map), compose(f.mor_map, g.mor_map))
} by {
    if f.src_cat = g.dst_cat {
        forall(m: M1) {
            functor_src(g, m)
            functor_src(f, g.mor_map(m))
            f.dst_cat.src(compose(f.mor_map, g.mor_map, m)) = compose(f.obj_map, g.obj_map, g.src_cat.src(m))
        }
    }
}

/// The composition of two functors satisfies the target axiom.
theorem compose_functor_dst_axiom[O1, M1, O2, M2, O3, M3](
    f: Functor[O2, M2, O3, M3], g: Functor[O1, M1, O2, M2]
) {
    f.src_cat = g.dst_cat implies
        functor_dst_axiom(g.src_cat, f.dst_cat, compose(f.obj_map, g.obj_map), compose(f.mor_map, g.mor_map))
} by {
    if f.src_cat = g.dst_cat {
        forall(m: M1) {
            functor_dst(g, m)
            functor_dst(f, g.mor_map(m))
            f.dst_cat.dst(compose(f.mor_map, g.mor_map, m)) = compose(f.obj_map, g.obj_map, g.src_cat.dst(m))
        }
    }
}

/// The composition of two functors satisfies the identity axiom.
theorem compose_functor_identity_axiom[O1, M1, O2, M2, O3, M3](
    f: Functor[O2, M2, O3, M3], g: Functor[O1, M1, O2, M2]
) {
    f.src_cat = g.dst_cat implies
        functor_identity_axiom(g.src_cat, f.dst_cat, compose(f.obj_map, g.obj_map), compose(f.mor_map, g.mor_map))
} by {
    if f.src_cat = g.dst_cat {
        forall(x: O1) {
            functor_identity(g, x)
            functor_identity(f, g.obj_map(x))
            compose(f.mor_map, g.mor_map, g.src_cat.identity(x)) = f.dst_cat.identity(compose(f.obj_map, g.obj_map, x))
        }
    }
}

/// The composition of two functors satisfies the composition axiom.
theorem compose_functor_compose_axiom[O1, M1, O2, M2, O3, M3](
    f: Functor[O2, M2, O3, M3], g: Functor[O1, M1, O2, M2]
) {
    f.src_cat = g.dst_cat implies
        functor_compose_axiom(g.src_cat, f.dst_cat, compose(f.obj_map, g.obj_map), compose(f.mor_map, g.mor_map))
} by {
    if f.src_cat = g.dst_cat {
        forall(p: M1, q: M1) {
            if g.src_cat.src(p) = g.src_cat.dst(q) {
                functor_compose(g, p, q)
                functor_dst(g, q)
                functor_src(g, p)
                f.src_cat.src(g.mor_map(p)) = f.src_cat.dst(g.mor_map(q))
                functor_compose(f, g.mor_map(p), g.mor_map(q))
                compose(f.mor_map, g.mor_map, g.src_cat.compose(p, q)) = f.dst_cat.compose(compose(f.mor_map, g.mor_map, p), compose(f.mor_map, g.mor_map, q))
            }
        }
    }
}

/// The componentwise composition of two functors is a functor.
theorem compose_is_functor[O1, M1, O2, M2, O3, M3](
    f: Functor[O2, M2, O3, M3], g: Functor[O1, M1, O2, M2]
) {
    f.src_cat = g.dst_cat implies
        is_functor(g.src_cat, f.dst_cat, compose(f.obj_map, g.obj_map), compose(f.mor_map, g.mor_map))
} by {
    if f.src_cat = g.dst_cat {
        compose_functor_src_axiom(f, g)
        compose_functor_dst_axiom(f, g)
        compose_functor_identity_axiom(f, g)
        compose_functor_compose_axiom(f, g)
        (functor_src_axiom(g.src_cat, f.dst_cat, compose(f.obj_map, g.obj_map), compose(f.mor_map, g.mor_map))
            and functor_dst_axiom(g.src_cat, f.dst_cat, compose(f.obj_map, g.obj_map), compose(f.mor_map, g.mor_map))
            and functor_identity_axiom(g.src_cat, f.dst_cat, compose(f.obj_map, g.obj_map), compose(f.mor_map, g.mor_map))
            and functor_compose_axiom(g.src_cat, f.dst_cat, compose(f.obj_map, g.obj_map), compose(f.mor_map, g.mor_map)))
    }
}

/// The componentwise composition of two functors, packaged as an optional functor.
/// Returns `Option.some(...)` exactly when the source category of `f` matches the target category of `g`.
define compose_functor[O1, M1, O2, M2, O3, M3](
    f: Functor[O2, M2, O3, M3], g: Functor[O1, M1, O2, M2]
) -> Option[Functor[O1, M1, O3, M3]] {
    Functor[O1, M1, O3, M3].new(
        g.src_cat, f.dst_cat,
        compose(f.obj_map, g.obj_map),
        compose(f.mor_map, g.mor_map)
    )
}

/// The composition of two composable functors yields some bundled functor.
theorem compose_functor_some[O1, M1, O2, M2, O3, M3](
    f: Functor[O2, M2, O3, M3], g: Functor[O1, M1, O2, M2]
) {
    f.src_cat = g.dst_cat implies exists(h: Functor[O1, M1, O3, M3]) {
        compose_functor(f, g) = Option.some(h)
    }
} by {
    if f.src_cat = g.dst_cat {
        compose_is_functor(f, g)
        let h: Functor[O1, M1, O3, M3] satisfy {
            Functor[O1, M1, O3, M3].new(
                g.src_cat, f.dst_cat,
                compose(f.obj_map, g.obj_map),
                compose(f.mor_map, g.mor_map)
            ) = Option.some(h)
        }
        compose_functor(f, g) = Option.some(h)
    }
}

/// A bundled composite functor inherits the source category of the inner functor.
theorem compose_functor_src_cat[O1, M1, O2, M2, O3, M3](
    f: Functor[O2, M2, O3, M3], g: Functor[O1, M1, O2, M2],
    h: Functor[O1, M1, O3, M3]
) {
    compose_functor(f, g) = Option.some(h) implies h.src_cat = g.src_cat
} by {
    if compose_functor(f, g) = Option.some(h) {
        Functor[O1, M1, O3, M3].new(
            g.src_cat, f.dst_cat,
            compose(f.obj_map, g.obj_map),
            compose(f.mor_map, g.mor_map)
        ) = Option.some(h)
        functor_new_src_cat(
            g.src_cat, f.dst_cat,
            compose(f.obj_map, g.obj_map),
            compose(f.mor_map, g.mor_map),
            h
        )
    }
}

/// A bundled composite functor inherits the target category of the outer functor.
theorem compose_functor_dst_cat[O1, M1, O2, M2, O3, M3](
    f: Functor[O2, M2, O3, M3], g: Functor[O1, M1, O2, M2],
    h: Functor[O1, M1, O3, M3]
) {
    compose_functor(f, g) = Option.some(h) implies h.dst_cat = f.dst_cat
} by {
    if compose_functor(f, g) = Option.some(h) {
        Functor[O1, M1, O3, M3].new(
            g.src_cat, f.dst_cat,
            compose(f.obj_map, g.obj_map),
            compose(f.mor_map, g.mor_map)
        ) = Option.some(h)
        functor_new_dst_cat(
            g.src_cat, f.dst_cat,
            compose(f.obj_map, g.obj_map),
            compose(f.mor_map, g.mor_map),
            h
        )
    }
}

/// The action on objects of a bundled composite functor is the composition of the underlying object maps.
theorem compose_functor_obj_map[O1, M1, O2, M2, O3, M3](
    f: Functor[O2, M2, O3, M3], g: Functor[O1, M1, O2, M2],
    h: Functor[O1, M1, O3, M3]
) {
    compose_functor(f, g) = Option.some(h) implies h.obj_map = compose(f.obj_map, g.obj_map)
} by {
    if compose_functor(f, g) = Option.some(h) {
        Functor[O1, M1, O3, M3].new(
            g.src_cat, f.dst_cat,
            compose(f.obj_map, g.obj_map),
            compose(f.mor_map, g.mor_map)
        ) = Option.some(h)
        functor_new_obj_map(
            g.src_cat, f.dst_cat,
            compose(f.obj_map, g.obj_map),
            compose(f.mor_map, g.mor_map),
            h
        )
    }
}

/// The action on morphisms of a bundled composite functor is the composition of the underlying morphism maps.
theorem compose_functor_mor_map[O1, M1, O2, M2, O3, M3](
    f: Functor[O2, M2, O3, M3], g: Functor[O1, M1, O2, M2],
    h: Functor[O1, M1, O3, M3]
) {
    compose_functor(f, g) = Option.some(h) implies h.mor_map = compose(f.mor_map, g.mor_map)
} by {
    if compose_functor(f, g) = Option.some(h) {
        Functor[O1, M1, O3, M3].new(
            g.src_cat, f.dst_cat,
            compose(f.obj_map, g.obj_map),
            compose(f.mor_map, g.mor_map)
        ) = Option.some(h)
        functor_new_mor_map(
            g.src_cat, f.dst_cat,
            compose(f.obj_map, g.obj_map),
            compose(f.mor_map, g.mor_map),
            h
        )
    }
}

/// Composing the identity functor on the target on the left with a functor recovers the original functor.
theorem compose_functor_identity_left[O1, M1, O2, M2](f: Functor[O1, M1, O2, M2]) {
    compose_functor(identity_functor(f.dst_cat), f) = Option.some(f)
} by {
    let i = identity_functor(f.dst_cat)
    identity_functor_src_cat(f.dst_cat)
    identity_functor_dst_cat(f.dst_cat)
    identity_functor_obj_map(f.dst_cat)
    identity_functor_mor_map(f.dst_cat)
    compose_functor_some(i, f)
    let h: Functor[O1, M1, O2, M2] satisfy {
        compose_functor(i, f) = Option.some(h)
    }
    compose_functor_src_cat(i, f, h)
    compose_functor_dst_cat(i, f, h)
    compose_functor_obj_map(i, f, h)
    compose_identity_left(f.obj_map)
    h.obj_map = f.obj_map
    compose_functor_mor_map(i, f, h)
    compose_identity_left(f.mor_map)
    h.mor_map = f.mor_map
    functor_ext(h, f)
}

/// Composing a functor on the left with the identity functor on its source recovers the original functor.
theorem compose_functor_identity_right[O1, M1, O2, M2](f: Functor[O1, M1, O2, M2]) {
    compose_functor(f, identity_functor(f.src_cat)) = Option.some(f)
} by {
    let i = identity_functor(f.src_cat)
    identity_functor_src_cat(f.src_cat)
    identity_functor_dst_cat(f.src_cat)
    identity_functor_obj_map(f.src_cat)
    identity_functor_mor_map(f.src_cat)
    compose_functor_some(f, i)
    let h: Functor[O1, M1, O2, M2] satisfy {
        compose_functor(f, i) = Option.some(h)
    }
    compose_functor_src_cat(f, i, h)
    compose_functor_dst_cat(f, i, h)
    compose_functor_obj_map(f, i, h)
    compose_identity_right(f.obj_map)
    h.obj_map = f.obj_map
    compose_functor_mor_map(f, i, h)
    compose_identity_right(f.mor_map)
    h.mor_map = f.mor_map
    functor_ext(h, f)
}

/// The left identity functor is composable with any functor targeting its category.
theorem compose_functor_identity_left_composable[O1, M1, O2, M2](f: Functor[O1, M1, O2, M2]) {
    identity_functor(f.dst_cat).src_cat = f.dst_cat
} by {
    identity_functor_src_cat(f.dst_cat)
}

/// Any functor is composable with the right identity functor on its source category.
theorem compose_functor_identity_right_composable[O1, M1, O2, M2](f: Functor[O1, M1, O2, M2]) {
    f.src_cat = identity_functor(f.src_cat).dst_cat
} by {
    identity_functor_dst_cat(f.src_cat)
}

/// The source category of a left identity composite is the original source category.
theorem compose_functor_identity_left_src_cat[O1, M1, O2, M2](
    f: Functor[O1, M1, O2, M2], h: Functor[O1, M1, O2, M2]
) {
    compose_functor(identity_functor(f.dst_cat), f) = Option.some(h) implies h.src_cat = f.src_cat
} by {
    if compose_functor(identity_functor(f.dst_cat), f) = Option.some(h) {
        compose_functor_src_cat(identity_functor(f.dst_cat), f, h)
    }
}

/// The target category of a left identity composite is the original target category.
theorem compose_functor_identity_left_dst_cat[O1, M1, O2, M2](
    f: Functor[O1, M1, O2, M2], h: Functor[O1, M1, O2, M2]
) {
    compose_functor(identity_functor(f.dst_cat), f) = Option.some(h) implies h.dst_cat = f.dst_cat
} by {
    if compose_functor(identity_functor(f.dst_cat), f) = Option.some(h) {
        compose_functor_dst_cat(identity_functor(f.dst_cat), f, h)
        h.dst_cat = identity_functor(f.dst_cat).dst_cat
        identity_functor_dst_cat(f.dst_cat)
    }
}

/// The object map of a left identity composite is the original object map.
theorem compose_functor_identity_left_obj_map[O1, M1, O2, M2](
    f: Functor[O1, M1, O2, M2], h: Functor[O1, M1, O2, M2]
) {
    compose_functor(identity_functor(f.dst_cat), f) = Option.some(h) implies h.obj_map = f.obj_map
} by {
    if compose_functor(identity_functor(f.dst_cat), f) = Option.some(h) {
        compose_functor_obj_map(identity_functor(f.dst_cat), f, h)
        h.obj_map = compose(identity_functor(f.dst_cat).obj_map, f.obj_map)
        identity_functor_obj_map(f.dst_cat)
        identity_functor(f.dst_cat).obj_map = identity_fn[O2]
        h.obj_map = compose(identity_fn[O2], f.obj_map)
        compose_identity_left(f.obj_map)
    }
}

/// The morphism map of a left identity composite is the original morphism map.
theorem compose_functor_identity_left_mor_map[O1, M1, O2, M2](
    f: Functor[O1, M1, O2, M2], h: Functor[O1, M1, O2, M2]
) {
    compose_functor(identity_functor(f.dst_cat), f) = Option.some(h) implies h.mor_map = f.mor_map
} by {
    if compose_functor(identity_functor(f.dst_cat), f) = Option.some(h) {
        compose_functor_mor_map(identity_functor(f.dst_cat), f, h)
        h.mor_map = compose(identity_functor(f.dst_cat).mor_map, f.mor_map)
        identity_functor_mor_map(f.dst_cat)
        identity_functor(f.dst_cat).mor_map = identity_fn[M2]
        h.mor_map = compose(identity_fn[M2], f.mor_map)
        compose_identity_left(f.mor_map)
    }
}

/// The source category of a right identity composite is the original source category.
theorem compose_functor_identity_right_src_cat[O1, M1, O2, M2](
    f: Functor[O1, M1, O2, M2], h: Functor[O1, M1, O2, M2]
) {
    compose_functor(f, identity_functor(f.src_cat)) = Option.some(h) implies h.src_cat = f.src_cat
} by {
    if compose_functor(f, identity_functor(f.src_cat)) = Option.some(h) {
        compose_functor_src_cat(f, identity_functor(f.src_cat), h)
        h.src_cat = identity_functor(f.src_cat).src_cat
        identity_functor_src_cat(f.src_cat)
    }
}

/// The target category of a right identity composite is the original target category.
theorem compose_functor_identity_right_dst_cat[O1, M1, O2, M2](
    f: Functor[O1, M1, O2, M2], h: Functor[O1, M1, O2, M2]
) {
    compose_functor(f, identity_functor(f.src_cat)) = Option.some(h) implies h.dst_cat = f.dst_cat
} by {
    if compose_functor(f, identity_functor(f.src_cat)) = Option.some(h) {
        compose_functor_dst_cat(f, identity_functor(f.src_cat), h)
    }
}

/// The object map of a right identity composite is the original object map.
theorem compose_functor_identity_right_obj_map[O1, M1, O2, M2](
    f: Functor[O1, M1, O2, M2], h: Functor[O1, M1, O2, M2]
) {
    compose_functor(f, identity_functor(f.src_cat)) = Option.some(h) implies h.obj_map = f.obj_map
} by {
    if compose_functor(f, identity_functor(f.src_cat)) = Option.some(h) {
        compose_functor_obj_map(f, identity_functor(f.src_cat), h)
        h.obj_map = compose(f.obj_map, identity_functor(f.src_cat).obj_map)
        identity_functor_obj_map(f.src_cat)
        identity_functor(f.src_cat).obj_map = identity_fn[O1]
        h.obj_map = compose(f.obj_map, identity_fn[O1])
        compose_identity_right(f.obj_map)
    }
}

/// The morphism map of a right identity composite is the original morphism map.
theorem compose_functor_identity_right_mor_map[O1, M1, O2, M2](
    f: Functor[O1, M1, O2, M2], h: Functor[O1, M1, O2, M2]
) {
    compose_functor(f, identity_functor(f.src_cat)) = Option.some(h) implies h.mor_map = f.mor_map
} by {
    if compose_functor(f, identity_functor(f.src_cat)) = Option.some(h) {
        compose_functor_mor_map(f, identity_functor(f.src_cat), h)
        h.mor_map = compose(f.mor_map, identity_functor(f.src_cat).mor_map)
        identity_functor_mor_map(f.src_cat)
        identity_functor(f.src_cat).mor_map = identity_fn[M1]
        h.mor_map = compose(f.mor_map, identity_fn[M1])
        compose_identity_right(f.mor_map)
    }
}

/// The action on objects of the constant functor sending everything to `x0`.
define constant_obj_map[O1, O2](x0: O2, x: O1) -> O2 {
    x0
}

/// The action on morphisms of the constant functor at `x0`: every morphism goes to the identity at `x0`.
define constant_mor_map[M1, O2, M2](d: Category[O2, M2], x0: O2, f: M1) -> M2 {
    d.identity(x0)
}

/// The constant maps at `x0` form a functor from `c` to `d`.
theorem constant_is_functor[O1, M1, O2, M2](
    c: Category[O1, M1], d: Category[O2, M2], x0: O2
) {
    is_functor(c, d, function(x: O1) { constant_obj_map(x0, x) },
                     function(f: M1) { constant_mor_map(d, x0, f) })
} by {
    let om: O1 -> O2 = function(x: O1) { constant_obj_map(x0, x) }
    let mm: M1 -> M2 = function(f: M1) { constant_mor_map(d, x0, f) }
    forall(x: O1) {
        om(x) = x0
    }
    forall(f: M1) {
        mm(f) = d.identity(x0)
    }
    forall(f: M1) {
        category_identity_src(d, x0)
        d.src(mm(f)) = om(c.src(f))
    }
    functor_src_axiom(c, d, om, mm)
    forall(f: M1) {
        category_identity_dst(d, x0)
        d.dst(mm(f)) = om(c.dst(f))
    }
    functor_dst_axiom(c, d, om, mm)
    forall(x: O1) {
        mm(c.identity(x)) = d.identity(om(x))
    }
    functor_identity_axiom(c, d, om, mm)
    forall(p: M1, q: M1) {
        if c.src(p) = c.dst(q) {
            category_identity_src(d, x0)
            category_identity_dst(d, x0)
            category_id_left(d, d.identity(x0))
            mm(c.compose(p, q)) = d.compose(mm(p), mm(q))
        }
    }
    functor_compose_axiom(c, d, om, mm)
    is_functor(c, d, om, mm)
}

/// The constant functor at `x0`, sending every object to `x0` and every morphism to the identity at `x0`.
let constant_functor[O1, M1, O2, M2](c: Category[O1, M1], d: Category[O2, M2], x0: O2) -> result: Functor[O1, M1, O2, M2] satisfy {
    Functor[O1, M1, O2, M2].new(c, d,
        function(x: O1) { constant_obj_map(x0, x) },
        function(f: M1) { constant_mor_map(d, x0, f) }) = Option.some(result)
} by {
    constant_is_functor(c, d, x0)
}

/// The constant functor has `c` as its source category.
theorem constant_functor_src_cat[O1, M1, O2, M2](
    c: Category[O1, M1], d: Category[O2, M2], x0: O2
) {
    constant_functor(c, d, x0).src_cat = c
} by {
    let f = constant_functor(c, d, x0)
    (Functor[O1, M1, O2, M2].new(c, d,
        function(x: O1) { constant_obj_map(x0, x) },
        function(g: M1) { constant_mor_map(d, x0, g) }) = Option.some(f))
    functor_new_src_cat(c, d,
        function(x: O1) { constant_obj_map(x0, x) },
        function(g: M1) { constant_mor_map(d, x0, g) }, f)
}

/// The constant functor has `d` as its target category.
theorem constant_functor_dst_cat[O1, M1, O2, M2](
    c: Category[O1, M1], d: Category[O2, M2], x0: O2
) {
    constant_functor(c, d, x0).dst_cat = d
} by {
    let f = constant_functor(c, d, x0)
    (Functor[O1, M1, O2, M2].new(c, d,
        function(x: O1) { constant_obj_map(x0, x) },
        function(g: M1) { constant_mor_map(d, x0, g) }) = Option.some(f))
    functor_new_dst_cat(c, d,
        function(x: O1) { constant_obj_map(x0, x) },
        function(g: M1) { constant_mor_map(d, x0, g) }, f)
}

/// The constant functor's action on objects is the constant map at `x0`.
theorem constant_functor_obj_map[O1, M1, O2, M2](
    c: Category[O1, M1], d: Category[O2, M2], x0: O2
) {
    constant_functor(c, d, x0).obj_map = function(x: O1) { constant_obj_map(x0, x) }
} by {
    let f = constant_functor(c, d, x0)
    (Functor[O1, M1, O2, M2].new(c, d,
        function(x: O1) { constant_obj_map(x0, x) },
        function(g: M1) { constant_mor_map(d, x0, g) }) = Option.some(f))
    functor_new_obj_map(c, d,
        function(x: O1) { constant_obj_map(x0, x) },
        function(g: M1) { constant_mor_map(d, x0, g) }, f)
}

/// The constant functor's action on morphisms sends each morphism to the identity at `x0`.
theorem constant_functor_mor_map[O1, M1, O2, M2](
    c: Category[O1, M1], d: Category[O2, M2], x0: O2
) {
    constant_functor(c, d, x0).mor_map = function(f: M1) { constant_mor_map(d, x0, f) }
} by {
    let f = constant_functor(c, d, x0)
    (Functor[O1, M1, O2, M2].new(c, d,
        function(x: O1) { constant_obj_map(x0, x) },
        function(g: M1) { constant_mor_map(d, x0, g) }) = Option.some(f))
    functor_new_mor_map(c, d,
        function(x: O1) { constant_obj_map(x0, x) },
        function(g: M1) { constant_mor_map(d, x0, g) }, f)
}

/// The constant functor sends every object to `x0`.
theorem constant_functor_obj_eq[O1, M1, O2, M2](
    c: Category[O1, M1], d: Category[O2, M2], x0: O2, x: O1
) {
    constant_functor(c, d, x0).obj_map(x) = x0
} by {
    constant_functor_obj_map(c, d, x0)
}

/// The constant functor sends every morphism to the identity at `x0`.
theorem constant_functor_mor_eq[O1, M1, O2, M2](
    c: Category[O1, M1], d: Category[O2, M2], x0: O2, f: M1
) {
    constant_functor(c, d, x0).mor_map(f) = d.identity(x0)
} by {
    constant_functor_mor_map(c, d, x0)
}

/// Composing a functor on the right with a constant functor gives a constant functor at the image.
theorem compose_functor_constant_right[O1, M1, O2, M2, O3, M3](
    f: Functor[O2, M2, O3, M3], c: Category[O1, M1], x0: O2
) {
    compose_functor(f, constant_functor(c, f.src_cat, x0))
        = Option.some(constant_functor(c, f.dst_cat, f.obj_map(x0)))
} by {
    let g = constant_functor(c, f.src_cat, x0)
    constant_functor_src_cat(c, f.src_cat, x0)
    g.src_cat = c
    constant_functor_dst_cat(c, f.src_cat, x0)
    g.dst_cat = f.src_cat
    f.src_cat = g.dst_cat
    compose_functor_some(f, g)
    let h: Functor[O1, M1, O3, M3] satisfy {
        compose_functor(f, g) = Option.some(h)
    }
    compose_functor_src_cat(f, g, h)
    h.src_cat = g.src_cat
    h.src_cat = c
    compose_functor_dst_cat(f, g, h)
    h.dst_cat = f.dst_cat
    compose_functor_obj_map(f, g, h)
    h.obj_map = compose(f.obj_map, g.obj_map)
    compose_functor_mor_map(f, g, h)
    h.mor_map = compose(f.mor_map, g.mor_map)

    let k = constant_functor(c, f.dst_cat, f.obj_map(x0))
    constant_functor_src_cat(c, f.dst_cat, f.obj_map(x0))
    k.src_cat = c
    constant_functor_dst_cat(c, f.dst_cat, f.obj_map(x0))
    k.dst_cat = f.dst_cat

    forall(x: O1) {
        constant_functor_obj_eq(c, f.src_cat, x0, x)
        constant_functor_obj_eq(c, f.dst_cat, f.obj_map(x0), x)
        compose(f.obj_map, g.obj_map)(x) = k.obj_map(x)
        h.obj_map(x) = k.obj_map(x)
    }
    function_extensionality(h.obj_map, k.obj_map)
    h.obj_map = k.obj_map

    forall(m: M1) {
        constant_functor_mor_eq(c, f.src_cat, x0, m)
        functor_identity(f, x0)
        constant_functor_mor_eq(c, f.dst_cat, f.obj_map(x0), m)
        compose(f.mor_map, g.mor_map)(m) = k.mor_map(m)
        h.mor_map(m) = k.mor_map(m)
    }
    function_extensionality(h.mor_map, k.mor_map)
    h.mor_map = k.mor_map

    functor_ext(h, k)
}

/// Composing a constant functor on the left with any functor gives the same constant functor on the smaller source.
theorem compose_functor_constant_left[O1, M1, O2, M2, O3, M3](
    f: Functor[O1, M1, O2, M2], e: Category[O3, M3], x0: O3
) {
    compose_functor(constant_functor(f.dst_cat, e, x0), f)
        = Option.some(constant_functor(f.src_cat, e, x0))
} by {
    let g = constant_functor(f.dst_cat, e, x0)
    constant_functor_src_cat(f.dst_cat, e, x0)
    g.src_cat = f.dst_cat
    constant_functor_dst_cat(f.dst_cat, e, x0)
    g.dst_cat = e
    compose_functor_some(g, f)
    let h: Functor[O1, M1, O3, M3] satisfy {
        compose_functor(g, f) = Option.some(h)
    }
    compose_functor_src_cat(g, f, h)
    h.src_cat = f.src_cat
    compose_functor_dst_cat(g, f, h)
    h.dst_cat = g.dst_cat
    h.dst_cat = e
    compose_functor_obj_map(g, f, h)
    h.obj_map = compose(g.obj_map, f.obj_map)
    compose_functor_mor_map(g, f, h)
    h.mor_map = compose(g.mor_map, f.mor_map)

    let k = constant_functor(f.src_cat, e, x0)
    constant_functor_src_cat(f.src_cat, e, x0)
    k.src_cat = f.src_cat
    constant_functor_dst_cat(f.src_cat, e, x0)
    k.dst_cat = e

    forall(x: O1) {
        constant_functor_obj_eq(f.dst_cat, e, x0, f.obj_map(x))
        constant_functor_obj_eq(f.src_cat, e, x0, x)
        h.obj_map(x) = k.obj_map(x)
    }
    function_extensionality(h.obj_map, k.obj_map)
    h.obj_map = k.obj_map

    forall(m: M1) {
        constant_functor_mor_eq(f.dst_cat, e, x0, f.mor_map(m))
        constant_functor_mor_eq(f.src_cat, e, x0, m)
        h.mor_map(m) = k.mor_map(m)
    }
    function_extensionality(h.mor_map, k.mor_map)
    h.mor_map = k.mor_map

    functor_ext(h, k)
}
