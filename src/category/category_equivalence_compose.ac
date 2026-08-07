/// Composition boundary facts for equivalences of categories.

from category.category import category_id_right
from category.category_iso import is_iso_pair
from category.functor import Functor, functor_identity, compose_functor,
    compose_functor_src_cat, compose_functor_dst_cat,
    compose_functor_obj_map, compose_functor_mor_map, compose_functor_some, functor_ext
from category.functor_iso import functor_preserves_iso_pair
from category.natural_transformation import NaturalTransformation, identity_nat_trans,
    identity_nat_trans_src_functor, identity_nat_trans_dst_functor,
    identity_nat_trans_component_eq, identity_nat_trans_component,
    horizontal_composable, horizontal_compose_component,
    horizontal_compose_identity_outer_component_at, horizontal_compose_nat_trans,
    horizontal_compose_nat_trans_some,
    horizontal_compose_nat_trans_src_functor, horizontal_compose_nat_trans_dst_functor,
    horizontal_compose_nat_trans_component, nat_trans_shared_src_cat,
    nat_trans_shared_dst_cat, nat_trans_component_src
from category.natural_isomorphism import is_natural_isomorphism_pair,
    is_natural_isomorphism_pair_src_eq, is_natural_isomorphism_pair_dst_eq,
    is_natural_isomorphism_pair_components, nat_iso_components_inverse_at
from data.basic.functions import compose, compose_assoc

/// Horizontally composing an identity natural transformation on the right gives right whiskering on components.
theorem horizontal_compose_identity_inner_component_at[O1, M1, O2, M2, O3, M3](
    outer: NaturalTransformation[O2, M2, O3, M3],
    h: Functor[O1, M1, O2, M2],
    x: O1
) {
    horizontal_composable(outer, identity_nat_trans(h)) implies
        horizontal_compose_component(outer, identity_nat_trans(h), x) = outer.component(h.obj_map(x))
} by {
    let id_h = identity_nat_trans(h)
    if horizontal_composable(outer, identity_nat_trans(h)) {
        horizontal_composable(outer, id_h)
        identity_nat_trans_src_functor(h)
        id_h.src_functor = h
        identity_nat_trans_dst_functor(h)
        id_h.dst_functor = h
        identity_nat_trans_component_eq(h)
        id_h.component = identity_nat_trans_component(h)
        identity_nat_trans_component(h)(x) = identity_nat_trans_component(h, x)

        outer.src_functor.src_cat.identity(h.obj_map(x)) = h.dst_cat.identity(h.obj_map(x))
        functor_identity(outer.src_functor, h.obj_map(x))

        nat_trans_component_src(outer, h.obj_map(x))
        category_id_right(outer.src_functor.dst_cat, outer.component(h.obj_map(x)))
        horizontal_compose_component(outer, id_h, x) = outer.component(h.obj_map(x))
    }
}

/// Left whiskering preserves natural-isomorphism pairs.
theorem equivalence_compose_left_whisker_natural_isomorphism_pair[O1, M1, O2, M2, O3, M3](
    h: Functor[O2, M2, O3, M3],
    fwd: NaturalTransformation[O1, M1, O2, M2],
    bwd: NaturalTransformation[O1, M1, O2, M2],
    src_comp: Functor[O1, M1, O3, M3],
    dst_comp: Functor[O1, M1, O3, M3],
    whiskered_fwd: NaturalTransformation[O1, M1, O3, M3],
    whiskered_bwd: NaturalTransformation[O1, M1, O3, M3]
) {
    is_natural_isomorphism_pair(fwd, bwd)
    and h.src_cat = fwd.src_functor.dst_cat
    and compose_functor(h, fwd.src_functor) = Option.some(src_comp)
    and compose_functor(h, fwd.dst_functor) = Option.some(dst_comp)
    and horizontal_compose_nat_trans(identity_nat_trans(h), fwd, src_comp, dst_comp) = Option.some(whiskered_fwd)
    and horizontal_compose_nat_trans(identity_nat_trans(h), bwd, dst_comp, src_comp) = Option.some(whiskered_bwd)
    implies is_natural_isomorphism_pair(whiskered_fwd, whiskered_bwd)
} by {
    let id_h = identity_nat_trans(h)
    if is_natural_isomorphism_pair(fwd, bwd)
    and h.src_cat = fwd.src_functor.dst_cat
    and compose_functor(h, fwd.src_functor) = Option.some(src_comp)
    and compose_functor(h, fwd.dst_functor) = Option.some(dst_comp)
    and horizontal_compose_nat_trans(identity_nat_trans(h), fwd, src_comp, dst_comp) = Option.some(whiskered_fwd)
    and horizontal_compose_nat_trans(identity_nat_trans(h), bwd, dst_comp, src_comp) = Option.some(whiskered_bwd) {
        horizontal_compose_nat_trans_src_functor(id_h, fwd, src_comp, dst_comp, whiskered_fwd)
        whiskered_fwd.src_functor = src_comp
        horizontal_compose_nat_trans_dst_functor(id_h, fwd, src_comp, dst_comp, whiskered_fwd)
        whiskered_fwd.dst_functor = dst_comp
        horizontal_compose_nat_trans_component(id_h, fwd, src_comp, dst_comp, whiskered_fwd)
        whiskered_fwd.component = horizontal_compose_component(id_h, fwd)
        horizontal_compose_nat_trans_src_functor(id_h, bwd, dst_comp, src_comp, whiskered_bwd)
        whiskered_bwd.src_functor = dst_comp
        horizontal_compose_nat_trans_dst_functor(id_h, bwd, dst_comp, src_comp, whiskered_bwd)
        whiskered_bwd.dst_functor = src_comp
        horizontal_compose_nat_trans_component(id_h, bwd, dst_comp, src_comp, whiskered_bwd)
        whiskered_bwd.component = horizontal_compose_component(id_h, bwd)
        whiskered_fwd.src_functor = whiskered_bwd.dst_functor
        whiskered_fwd.dst_functor = whiskered_bwd.src_functor

        identity_nat_trans_src_functor(h)
        id_h.src_functor = h
        horizontal_composable(id_h, fwd) = (id_h.src_functor.src_cat = fwd.src_functor.dst_cat)
        horizontal_composable(id_h, fwd)
        is_natural_isomorphism_pair_dst_eq(fwd, bwd)
        fwd.dst_functor = bwd.src_functor
        nat_trans_shared_dst_cat(fwd)
        fwd.src_functor.dst_cat = fwd.dst_functor.dst_cat
        h.src_cat = bwd.src_functor.dst_cat
        horizontal_composable(id_h, bwd) = (id_h.src_functor.src_cat = bwd.src_functor.dst_cat)
        horizontal_composable(id_h, bwd)
        compose_functor_dst_cat(h, fwd.src_functor, src_comp)
        src_comp.dst_cat = h.dst_cat
        whiskered_fwd.src_functor.dst_cat = h.dst_cat
        is_natural_isomorphism_pair_components(fwd, bwd)
        forall(x: O1) {
            nat_iso_components_inverse_at(fwd, bwd, x)
            is_iso_pair(fwd.src_functor.dst_cat, fwd.component(x), bwd.component(x))
            h.src_cat = fwd.src_functor.dst_cat
            is_iso_pair(h.src_cat, fwd.component(x), bwd.component(x))
            functor_preserves_iso_pair(h, fwd.component(x), bwd.component(x))
            is_iso_pair(h.dst_cat, h.mor_map(fwd.component(x)), h.mor_map(bwd.component(x)))
            horizontal_compose_identity_outer_component_at(h, fwd, x)
            horizontal_compose_component(id_h, fwd, x) = h.mor_map(fwd.component(x))
            whiskered_fwd.component(x) = horizontal_compose_component(id_h, fwd)(x)
            horizontal_compose_component(id_h, fwd)(x) = horizontal_compose_component(id_h, fwd, x)
            whiskered_fwd.component(x) = h.mor_map(fwd.component(x))
            horizontal_compose_identity_outer_component_at(h, bwd, x)
            horizontal_compose_component(id_h, bwd, x) = h.mor_map(bwd.component(x))
            horizontal_compose_component(id_h, bwd)(x) = horizontal_compose_component(id_h, bwd, x)
            is_iso_pair(whiskered_fwd.src_functor.dst_cat, whiskered_fwd.component(x), whiskered_bwd.component(x))
        }
        is_natural_isomorphism_pair(whiskered_fwd, whiskered_bwd) =
            (whiskered_fwd.src_functor = whiskered_bwd.dst_functor
            and whiskered_fwd.dst_functor = whiskered_bwd.src_functor
            and forall(x: O1) {
                is_iso_pair(whiskered_fwd.src_functor.dst_cat,
                    whiskered_fwd.component(x), whiskered_bwd.component(x))
            })
        is_natural_isomorphism_pair(whiskered_fwd, whiskered_bwd)
    }
}

/// A left-whiskered natural-isomorphism pair exists for supplied composite functors.
theorem equivalence_compose_left_whisker_natural_isomorphism_pair_exists[O1, M1, O2, M2, O3, M3](
    h: Functor[O2, M2, O3, M3],
    fwd: NaturalTransformation[O1, M1, O2, M2],
    bwd: NaturalTransformation[O1, M1, O2, M2],
    src_comp: Functor[O1, M1, O3, M3],
    dst_comp: Functor[O1, M1, O3, M3]
) {
    is_natural_isomorphism_pair(fwd, bwd)
    and h.src_cat = fwd.src_functor.dst_cat
    and compose_functor(h, fwd.src_functor) = Option.some(src_comp)
    and compose_functor(h, fwd.dst_functor) = Option.some(dst_comp)
    implies exists(
        whiskered_fwd: NaturalTransformation[O1, M1, O3, M3],
        whiskered_bwd: NaturalTransformation[O1, M1, O3, M3]
    ) {
        horizontal_compose_nat_trans(identity_nat_trans(h), fwd, src_comp, dst_comp) =
            Option.some(whiskered_fwd)
        and horizontal_compose_nat_trans(identity_nat_trans(h), bwd, dst_comp, src_comp) =
            Option.some(whiskered_bwd)
        and is_natural_isomorphism_pair(whiskered_fwd, whiskered_bwd)
    }
} by {
    let id_h = identity_nat_trans(h)
    if is_natural_isomorphism_pair(fwd, bwd)
    and h.src_cat = fwd.src_functor.dst_cat
    and compose_functor(h, fwd.src_functor) = Option.some(src_comp)
    and compose_functor(h, fwd.dst_functor) = Option.some(dst_comp) {
        identity_nat_trans_src_functor(h)
        id_h.src_functor = h
        identity_nat_trans_dst_functor(h)
        id_h.dst_functor = h
        horizontal_composable(id_h, fwd) = (id_h.src_functor.src_cat = fwd.src_functor.dst_cat)
        horizontal_composable(id_h, fwd)
        compose_functor(id_h.src_functor, fwd.src_functor) = Option.some(src_comp)
        compose_functor(id_h.dst_functor, fwd.dst_functor) = Option.some(dst_comp)
        (horizontal_composable(id_h, fwd)
            and compose_functor(id_h.src_functor, fwd.src_functor) = Option.some(src_comp)
            and compose_functor(id_h.dst_functor, fwd.dst_functor) = Option.some(dst_comp))
        horizontal_compose_nat_trans_some(id_h, fwd, src_comp, dst_comp)
        let whiskered_fwd: NaturalTransformation[O1, M1, O3, M3] satisfy {
            horizontal_compose_nat_trans(id_h, fwd, src_comp, dst_comp) = Option.some(whiskered_fwd)
        }

        is_natural_isomorphism_pair_src_eq(fwd, bwd)
        fwd.src_functor = bwd.dst_functor
        is_natural_isomorphism_pair_dst_eq(fwd, bwd)
        fwd.dst_functor = bwd.src_functor
        nat_trans_shared_dst_cat(fwd)
        fwd.src_functor.dst_cat = fwd.dst_functor.dst_cat
        h.src_cat = bwd.src_functor.dst_cat
        horizontal_composable(id_h, bwd) = (id_h.src_functor.src_cat = bwd.src_functor.dst_cat)
        horizontal_composable(id_h, bwd)
        compose_functor(h, bwd.src_functor) = Option.some(dst_comp)
        compose_functor(h, bwd.dst_functor) = Option.some(src_comp)
        compose_functor(id_h.src_functor, bwd.src_functor) = Option.some(dst_comp)
        compose_functor(id_h.dst_functor, bwd.dst_functor) = Option.some(src_comp)
        (horizontal_composable(id_h, bwd)
            and compose_functor(id_h.src_functor, bwd.src_functor) = Option.some(dst_comp)
            and compose_functor(id_h.dst_functor, bwd.dst_functor) = Option.some(src_comp))
        horizontal_compose_nat_trans_some(id_h, bwd, dst_comp, src_comp)
        let whiskered_bwd: NaturalTransformation[O1, M1, O3, M3] satisfy {
            horizontal_compose_nat_trans(id_h, bwd, dst_comp, src_comp) = Option.some(whiskered_bwd)
        }

        identity_nat_trans(h) = id_h
        horizontal_compose_nat_trans(identity_nat_trans(h), fwd, src_comp, dst_comp) =
            Option.some(whiskered_fwd)
        horizontal_compose_nat_trans(identity_nat_trans(h), bwd, dst_comp, src_comp) =
            Option.some(whiskered_bwd)
        (is_natural_isomorphism_pair(fwd, bwd)
            and h.src_cat = fwd.src_functor.dst_cat
            and compose_functor(h, fwd.src_functor) = Option.some(src_comp)
            and compose_functor(h, fwd.dst_functor) = Option.some(dst_comp)
            and horizontal_compose_nat_trans(identity_nat_trans(h), fwd, src_comp, dst_comp) = Option.some(whiskered_fwd)
            and horizontal_compose_nat_trans(identity_nat_trans(h), bwd, dst_comp, src_comp) = Option.some(whiskered_bwd))
        equivalence_compose_left_whisker_natural_isomorphism_pair(
            h, fwd, bwd, src_comp, dst_comp, whiskered_fwd, whiskered_bwd)
        is_natural_isomorphism_pair(whiskered_fwd, whiskered_bwd)
        exists(wf: NaturalTransformation[O1, M1, O3, M3], wb: NaturalTransformation[O1, M1, O3, M3]) {
            horizontal_compose_nat_trans(identity_nat_trans(h), fwd, src_comp, dst_comp) =
                Option.some(wf)
            and horizontal_compose_nat_trans(identity_nat_trans(h), bwd, dst_comp, src_comp) =
                Option.some(wb)
            and is_natural_isomorphism_pair(wf, wb)
        }
    }
}

/// Right whiskering preserves natural-isomorphism pairs.
theorem equivalence_compose_right_whisker_natural_isomorphism_pair[O1, M1, O2, M2, O3, M3](
    h: Functor[O1, M1, O2, M2],
    fwd: NaturalTransformation[O2, M2, O3, M3],
    bwd: NaturalTransformation[O2, M2, O3, M3],
    src_comp: Functor[O1, M1, O3, M3],
    dst_comp: Functor[O1, M1, O3, M3],
    whiskered_fwd: NaturalTransformation[O1, M1, O3, M3],
    whiskered_bwd: NaturalTransformation[O1, M1, O3, M3]
) {
    is_natural_isomorphism_pair(fwd, bwd)
    and h.dst_cat = fwd.src_functor.src_cat
    and compose_functor(fwd.src_functor, h) = Option.some(src_comp)
    and compose_functor(fwd.dst_functor, h) = Option.some(dst_comp)
    and horizontal_compose_nat_trans(fwd, identity_nat_trans(h), src_comp, dst_comp) = Option.some(whiskered_fwd)
    and horizontal_compose_nat_trans(bwd, identity_nat_trans(h), dst_comp, src_comp) = Option.some(whiskered_bwd)
    implies is_natural_isomorphism_pair(whiskered_fwd, whiskered_bwd)
} by {
    let id_h = identity_nat_trans(h)
    if is_natural_isomorphism_pair(fwd, bwd)
    and h.dst_cat = fwd.src_functor.src_cat
    and compose_functor(fwd.src_functor, h) = Option.some(src_comp)
    and compose_functor(fwd.dst_functor, h) = Option.some(dst_comp)
    and horizontal_compose_nat_trans(fwd, identity_nat_trans(h), src_comp, dst_comp) = Option.some(whiskered_fwd)
    and horizontal_compose_nat_trans(bwd, identity_nat_trans(h), dst_comp, src_comp) = Option.some(whiskered_bwd) {
        horizontal_compose_nat_trans_src_functor(fwd, id_h, src_comp, dst_comp, whiskered_fwd)
        whiskered_fwd.src_functor = src_comp
        horizontal_compose_nat_trans_dst_functor(fwd, id_h, src_comp, dst_comp, whiskered_fwd)
        whiskered_fwd.dst_functor = dst_comp
        horizontal_compose_nat_trans_component(fwd, id_h, src_comp, dst_comp, whiskered_fwd)
        whiskered_fwd.component = horizontal_compose_component(fwd, id_h)
        horizontal_compose_nat_trans_src_functor(bwd, id_h, dst_comp, src_comp, whiskered_bwd)
        whiskered_bwd.src_functor = dst_comp
        horizontal_compose_nat_trans_dst_functor(bwd, id_h, dst_comp, src_comp, whiskered_bwd)
        whiskered_bwd.dst_functor = src_comp
        horizontal_compose_nat_trans_component(bwd, id_h, dst_comp, src_comp, whiskered_bwd)
        whiskered_bwd.component = horizontal_compose_component(bwd, id_h)
        whiskered_fwd.src_functor = whiskered_bwd.dst_functor
        whiskered_fwd.dst_functor = whiskered_bwd.src_functor

        identity_nat_trans_src_functor(h)
        id_h.src_functor = h
        horizontal_composable(fwd, id_h) = (fwd.src_functor.src_cat = id_h.src_functor.dst_cat)
        horizontal_composable(fwd, id_h)
        is_natural_isomorphism_pair_dst_eq(fwd, bwd)
        fwd.dst_functor = bwd.src_functor
        nat_trans_shared_src_cat(fwd)
        fwd.src_functor.src_cat = fwd.dst_functor.src_cat
        h.dst_cat = bwd.src_functor.src_cat
        horizontal_composable(bwd, id_h) = (bwd.src_functor.src_cat = id_h.src_functor.dst_cat)
        horizontal_composable(bwd, id_h)
        compose_functor_dst_cat(fwd.src_functor, h, src_comp)
        src_comp.dst_cat = fwd.src_functor.dst_cat
        whiskered_fwd.src_functor.dst_cat = fwd.src_functor.dst_cat
        is_natural_isomorphism_pair_components(fwd, bwd)
        forall(x: O1) {
            nat_iso_components_inverse_at(fwd, bwd, h.obj_map(x))
            is_iso_pair(fwd.src_functor.dst_cat, fwd.component(h.obj_map(x)), bwd.component(h.obj_map(x)))
            horizontal_compose_identity_inner_component_at(fwd, h, x)
            horizontal_compose_component(fwd, id_h, x) = fwd.component(h.obj_map(x))
            whiskered_fwd.component(x) = horizontal_compose_component(fwd, id_h)(x)
            horizontal_compose_component(fwd, id_h)(x) = horizontal_compose_component(fwd, id_h, x)
            whiskered_fwd.component(x) = fwd.component(h.obj_map(x))
            horizontal_compose_identity_inner_component_at(bwd, h, x)
            horizontal_compose_component(bwd, id_h, x) = bwd.component(h.obj_map(x))
            horizontal_compose_component(bwd, id_h)(x) = horizontal_compose_component(bwd, id_h, x)
            is_iso_pair(whiskered_fwd.src_functor.dst_cat, whiskered_fwd.component(x), whiskered_bwd.component(x))
        }
        is_natural_isomorphism_pair(whiskered_fwd, whiskered_bwd) =
            (whiskered_fwd.src_functor = whiskered_bwd.dst_functor
            and whiskered_fwd.dst_functor = whiskered_bwd.src_functor
            and forall(x: O1) {
                is_iso_pair(whiskered_fwd.src_functor.dst_cat,
                    whiskered_fwd.component(x), whiskered_bwd.component(x))
            })
        is_natural_isomorphism_pair(whiskered_fwd, whiskered_bwd)
    }
}

/// A right-whiskered natural-isomorphism pair exists for supplied composite functors.
theorem equivalence_compose_right_whisker_natural_isomorphism_pair_exists[O1, M1, O2, M2, O3, M3](
    h: Functor[O1, M1, O2, M2],
    fwd: NaturalTransformation[O2, M2, O3, M3],
    bwd: NaturalTransformation[O2, M2, O3, M3],
    src_comp: Functor[O1, M1, O3, M3],
    dst_comp: Functor[O1, M1, O3, M3]
) {
    is_natural_isomorphism_pair(fwd, bwd)
    and h.dst_cat = fwd.src_functor.src_cat
    and compose_functor(fwd.src_functor, h) = Option.some(src_comp)
    and compose_functor(fwd.dst_functor, h) = Option.some(dst_comp)
    implies exists(
        whiskered_fwd: NaturalTransformation[O1, M1, O3, M3],
        whiskered_bwd: NaturalTransformation[O1, M1, O3, M3]
    ) {
        horizontal_compose_nat_trans(fwd, identity_nat_trans(h), src_comp, dst_comp) =
            Option.some(whiskered_fwd)
        and horizontal_compose_nat_trans(bwd, identity_nat_trans(h), dst_comp, src_comp) =
            Option.some(whiskered_bwd)
        and is_natural_isomorphism_pair(whiskered_fwd, whiskered_bwd)
    }
} by {
    let id_h = identity_nat_trans(h)
    if is_natural_isomorphism_pair(fwd, bwd)
    and h.dst_cat = fwd.src_functor.src_cat
    and compose_functor(fwd.src_functor, h) = Option.some(src_comp)
    and compose_functor(fwd.dst_functor, h) = Option.some(dst_comp) {
        identity_nat_trans_src_functor(h)
        id_h.src_functor = h
        identity_nat_trans_dst_functor(h)
        id_h.dst_functor = h
        horizontal_composable(fwd, id_h) = (fwd.src_functor.src_cat = id_h.src_functor.dst_cat)
        horizontal_composable(fwd, id_h)
        compose_functor(fwd.src_functor, id_h.src_functor) = Option.some(src_comp)
        compose_functor(fwd.dst_functor, id_h.dst_functor) = Option.some(dst_comp)
        (horizontal_composable(fwd, id_h)
            and compose_functor(fwd.src_functor, id_h.src_functor) = Option.some(src_comp)
            and compose_functor(fwd.dst_functor, id_h.dst_functor) = Option.some(dst_comp))
        horizontal_compose_nat_trans_some(fwd, id_h, src_comp, dst_comp)
        let whiskered_fwd: NaturalTransformation[O1, M1, O3, M3] satisfy {
            horizontal_compose_nat_trans(fwd, id_h, src_comp, dst_comp) = Option.some(whiskered_fwd)
        }

        is_natural_isomorphism_pair_src_eq(fwd, bwd)
        fwd.src_functor = bwd.dst_functor
        is_natural_isomorphism_pair_dst_eq(fwd, bwd)
        fwd.dst_functor = bwd.src_functor
        nat_trans_shared_src_cat(fwd)
        fwd.src_functor.src_cat = fwd.dst_functor.src_cat
        h.dst_cat = bwd.src_functor.src_cat
        horizontal_composable(bwd, id_h) = (bwd.src_functor.src_cat = id_h.src_functor.dst_cat)
        horizontal_composable(bwd, id_h)
        compose_functor(bwd.src_functor, h) = Option.some(dst_comp)
        compose_functor(bwd.dst_functor, h) = Option.some(src_comp)
        compose_functor(bwd.src_functor, id_h.src_functor) = Option.some(dst_comp)
        compose_functor(bwd.dst_functor, id_h.dst_functor) = Option.some(src_comp)
        (horizontal_composable(bwd, id_h)
            and compose_functor(bwd.src_functor, id_h.src_functor) = Option.some(dst_comp)
            and compose_functor(bwd.dst_functor, id_h.dst_functor) = Option.some(src_comp))
        horizontal_compose_nat_trans_some(bwd, id_h, dst_comp, src_comp)
        let whiskered_bwd: NaturalTransformation[O1, M1, O3, M3] satisfy {
            horizontal_compose_nat_trans(bwd, id_h, dst_comp, src_comp) = Option.some(whiskered_bwd)
        }

        identity_nat_trans(h) = id_h
        horizontal_compose_nat_trans(fwd, identity_nat_trans(h), src_comp, dst_comp) =
            Option.some(whiskered_fwd)
        horizontal_compose_nat_trans(bwd, identity_nat_trans(h), dst_comp, src_comp) =
            Option.some(whiskered_bwd)
        (is_natural_isomorphism_pair(fwd, bwd)
            and h.dst_cat = fwd.src_functor.src_cat
            and compose_functor(fwd.src_functor, h) = Option.some(src_comp)
            and compose_functor(fwd.dst_functor, h) = Option.some(dst_comp)
            and horizontal_compose_nat_trans(fwd, identity_nat_trans(h), src_comp, dst_comp) = Option.some(whiskered_fwd)
            and horizontal_compose_nat_trans(bwd, identity_nat_trans(h), dst_comp, src_comp) = Option.some(whiskered_bwd))
        equivalence_compose_right_whisker_natural_isomorphism_pair(
            h, fwd, bwd, src_comp, dst_comp, whiskered_fwd, whiskered_bwd)
        is_natural_isomorphism_pair(whiskered_fwd, whiskered_bwd)
        exists(wf: NaturalTransformation[O1, M1, O3, M3], wb: NaturalTransformation[O1, M1, O3, M3]) {
            horizontal_compose_nat_trans(fwd, identity_nat_trans(h), src_comp, dst_comp) =
                Option.some(wf)
            and horizontal_compose_nat_trans(bwd, identity_nat_trans(h), dst_comp, src_comp) =
                Option.some(wb)
            and is_natural_isomorphism_pair(wf, wb)
        }
    }
}

/// Bundled functor composition is associative for any supplied composites.
theorem compose_functor_assoc_eq[O1, M1, O2, M2, O3, M3, O4, M4](
    f: Functor[O3, M3, O4, M4],
    g: Functor[O2, M2, O3, M3],
    h: Functor[O1, M1, O2, M2],
    gh: Functor[O1, M1, O3, M3],
    fg: Functor[O2, M2, O4, M4],
    left: Functor[O1, M1, O4, M4],
    right: Functor[O1, M1, O4, M4]
) {
    compose_functor(g, h) = Option.some(gh)
    and compose_functor(f, g) = Option.some(fg)
    and compose_functor(f, gh) = Option.some(left)
    and compose_functor(fg, h) = Option.some(right)
    implies left = right
} by {
    if compose_functor(g, h) = Option.some(gh)
    and compose_functor(f, g) = Option.some(fg)
    and compose_functor(f, gh) = Option.some(left)
    and compose_functor(fg, h) = Option.some(right) {
        compose_functor_src_cat(g, h, gh)
        compose_functor_dst_cat(g, h, gh)
        compose_functor_src_cat(f, g, fg)
        compose_functor_dst_cat(f, g, fg)
        compose_functor_src_cat(f, gh, left)
        left.src_cat = h.src_cat
        compose_functor_dst_cat(f, gh, left)
        compose_functor_src_cat(fg, h, right)
        compose_functor_dst_cat(fg, h, right)
        right.dst_cat = f.dst_cat
        left.dst_cat = right.dst_cat

        compose_functor_obj_map(g, h, gh)
        compose_functor_obj_map(f, g, fg)
        compose_functor_obj_map(f, gh, left)
        compose_functor_obj_map(fg, h, right)
        compose_assoc(f.obj_map, g.obj_map, h.obj_map)
        left.obj_map = right.obj_map

        compose_functor_mor_map(g, h, gh)
        compose_functor_mor_map(f, g, fg)
        compose_functor_mor_map(f, gh, left)
        compose_functor_mor_map(fg, h, right)
        compose_assoc(f.mor_map, g.mor_map, h.mor_map)
        left.mor_map = right.mor_map

        functor_ext(left, right)
        left = right
    }
}

/// If `f` follows `g` and `g` follows `h`, then `f` follows the bundled composite `g ∘ h`.
theorem compose_functor_composable_left[O1, M1, O2, M2, O3, M3, O4, M4](
    f: Functor[O3, M3, O4, M4],
    g: Functor[O2, M2, O3, M3],
    h: Functor[O1, M1, O2, M2],
    gh: Functor[O1, M1, O3, M3]
) {
    g.src_cat = h.dst_cat and f.src_cat = g.dst_cat
    and compose_functor(g, h) = Option.some(gh)
    implies f.src_cat = gh.dst_cat
} by {
    if g.src_cat = h.dst_cat and f.src_cat = g.dst_cat
    and compose_functor(g, h) = Option.some(gh) {
        compose_functor_dst_cat(g, h, gh)
        gh.dst_cat = g.dst_cat
        f.src_cat = gh.dst_cat
    }
}

/// If `f` follows `g` and `g` follows `h`, then the bundled composite `f ∘ g` follows `h`.
theorem compose_functor_composable_right[O1, M1, O2, M2, O3, M3, O4, M4](
    f: Functor[O3, M3, O4, M4],
    g: Functor[O2, M2, O3, M3],
    h: Functor[O1, M1, O2, M2],
    fg: Functor[O2, M2, O4, M4]
) {
    g.src_cat = h.dst_cat and f.src_cat = g.dst_cat
    and compose_functor(f, g) = Option.some(fg)
    implies fg.src_cat = h.dst_cat
} by {
    if g.src_cat = h.dst_cat and f.src_cat = g.dst_cat
    and compose_functor(f, g) = Option.some(fg) {
        compose_functor_src_cat(f, g, fg)
        fg.src_cat = g.src_cat
        fg.src_cat = h.dst_cat
    }
}

/// A composable triple of functors has equal left- and right-associated bundled composites.
theorem compose_functor_assoc_some[O1, M1, O2, M2, O3, M3, O4, M4](
    f: Functor[O3, M3, O4, M4],
    g: Functor[O2, M2, O3, M3],
    h: Functor[O1, M1, O2, M2]
) {
    g.src_cat = h.dst_cat and f.src_cat = g.dst_cat implies
    exists(left: Functor[O1, M1, O4, M4], right: Functor[O1, M1, O4, M4]) {
        exists(gh: Functor[O1, M1, O3, M3], fg: Functor[O2, M2, O4, M4]) {
            compose_functor(g, h) = Option.some(gh)
            and compose_functor(f, g) = Option.some(fg)
            and compose_functor(f, gh) = Option.some(left)
            and compose_functor(fg, h) = Option.some(right)
            and left = right
        }
    }
} by {
    if g.src_cat = h.dst_cat and f.src_cat = g.dst_cat {
        compose_functor_some(g, h)
        let gh: Functor[O1, M1, O3, M3] satisfy {
            compose_functor(g, h) = Option.some(gh)
        }
        compose_functor_some(f, g)
        let fg: Functor[O2, M2, O4, M4] satisfy {
            compose_functor(f, g) = Option.some(fg)
        }
        compose_functor_composable_left(f, g, h, gh)
        f.src_cat = gh.dst_cat
        compose_functor_some(f, gh)
        let left: Functor[O1, M1, O4, M4] satisfy {
            compose_functor(f, gh) = Option.some(left)
        }
        compose_functor_composable_right(f, g, h, fg)
        fg.src_cat = h.dst_cat
        compose_functor_some(fg, h)
        let right: Functor[O1, M1, O4, M4] satisfy {
            compose_functor(fg, h) = Option.some(right)
        }
        compose_functor_assoc_eq(f, g, h, gh, fg, left, right)
        left = right
        exists(l: Functor[O1, M1, O4, M4], r: Functor[O1, M1, O4, M4]) {
            exists(a: Functor[O1, M1, O3, M3], b: Functor[O2, M2, O4, M4]) {
                compose_functor(g, h) = Option.some(a)
                and compose_functor(f, g) = Option.some(b)
                and compose_functor(f, a) = Option.some(l)
                and compose_functor(b, h) = Option.some(r)
                and l = r
            }
        }
    }
}
