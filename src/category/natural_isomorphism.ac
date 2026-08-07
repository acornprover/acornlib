/// Natural isomorphisms between functors.

from category.category_iso import is_iso_pair, is_iso_pair_identity, is_iso_pair_swap,
    is_iso_pair_compose, Iso, iso_new_cat, iso_new_hom, iso_new_inv,
    identity_iso, identity_iso_cat, identity_iso_hom, identity_iso_inv,
    inverse_iso, inverse_iso_cat, inverse_iso_hom, inverse_iso_inv, iso_ext,
    iso_src_hom, iso_src_inv
from category.functor import Functor
from category.natural_transformation import NaturalTransformation, identity_nat_trans,
    identity_nat_trans_src_functor, identity_nat_trans_dst_functor,
    identity_nat_trans_component_eq, identity_nat_trans_component,
    nat_trans_shared_dst_cat, nat_trans_component_src, nat_trans_component_dst,
    vertical_composable, vertical_compose_component, vertical_compose_nat_trans,
    vertical_compose_nat_trans_some, vertical_compose_nat_trans_src_functor,
    vertical_compose_nat_trans_dst_functor, vertical_compose_nat_trans_component

/// True if `forward` and `backward` are pointwise an inverse pair, at every object.
define nat_iso_components_inverse[O1, M1, O2, M2](
    forward: NaturalTransformation[O1, M1, O2, M2],
    backward: NaturalTransformation[O1, M1, O2, M2]
) -> Bool {
    forall(x: O1) {
        is_iso_pair(forward.src_functor.dst_cat, forward.component(x), backward.component(x))
    }
}

/// True if `forward` and `backward` form an inverse pair of natural transformations.
define is_natural_isomorphism_pair[O1, M1, O2, M2](
    forward: NaturalTransformation[O1, M1, O2, M2],
    backward: NaturalTransformation[O1, M1, O2, M2]
) -> Bool {
    forward.src_functor = backward.dst_functor
    and forward.dst_functor = backward.src_functor
    and nat_iso_components_inverse(forward, backward)
}

/// Extract the source-functor matching from the natural-isomorphism pair predicate.
theorem is_natural_isomorphism_pair_src_eq[O1, M1, O2, M2](
    forward: NaturalTransformation[O1, M1, O2, M2],
    backward: NaturalTransformation[O1, M1, O2, M2]
) {
    is_natural_isomorphism_pair(forward, backward) implies
        forward.src_functor = backward.dst_functor
}

/// Extract the target-functor matching from the natural-isomorphism pair predicate.
theorem is_natural_isomorphism_pair_dst_eq[O1, M1, O2, M2](
    forward: NaturalTransformation[O1, M1, O2, M2],
    backward: NaturalTransformation[O1, M1, O2, M2]
) {
    is_natural_isomorphism_pair(forward, backward) implies
        forward.dst_functor = backward.src_functor
}

/// Extract the components-inverse predicate from the natural-isomorphism pair predicate.
theorem is_natural_isomorphism_pair_components[O1, M1, O2, M2](
    forward: NaturalTransformation[O1, M1, O2, M2],
    backward: NaturalTransformation[O1, M1, O2, M2]
) {
    is_natural_isomorphism_pair(forward, backward) implies
        nat_iso_components_inverse(forward, backward)
}

/// Pointwise inverse pair from `nat_iso_components_inverse`.
theorem nat_iso_components_inverse_at[O1, M1, O2, M2](
    forward: NaturalTransformation[O1, M1, O2, M2],
    backward: NaturalTransformation[O1, M1, O2, M2], x: O1
) {
    nat_iso_components_inverse(forward, backward) implies
        is_iso_pair(forward.src_functor.dst_cat, forward.component(x), backward.component(x))
} by {
    if nat_iso_components_inverse(forward, backward) {
        nat_iso_components_inverse(forward, backward) = forall(y: O1) {
            is_iso_pair(forward.src_functor.dst_cat, forward.component(y), backward.component(y))
        }
    }
}

/// At each object, the identity natural transformation's component equals the identity morphism on the functor's image.
theorem identity_nat_trans_component_at[O1, M1, O2, M2](f: Functor[O1, M1, O2, M2], x: O1) {
    identity_nat_trans(f).component(x) = f.dst_cat.identity(f.obj_map(x))
} by {
    identity_nat_trans_component_eq(f)
}

/// The identity natural transformation's components form an inverse pair with themselves at each object.
theorem identity_nat_trans_iso_pair_at[O1, M1, O2, M2](f: Functor[O1, M1, O2, M2], x: O1) {
    is_iso_pair(f.dst_cat,
        identity_nat_trans(f).component(x),
        identity_nat_trans(f).component(x))
} by {
    identity_nat_trans_component_at(f, x)
    is_iso_pair_identity(f.dst_cat, f.obj_map(x))
}

/// The identity natural transformation has its functor as both source and target.
theorem identity_nat_trans_src_eq_dst[O1, M1, O2, M2](f: Functor[O1, M1, O2, M2]) {
    identity_nat_trans(f).src_functor = identity_nat_trans(f).dst_functor
} by {
    identity_nat_trans_src_functor(f)
    identity_nat_trans_dst_functor(f)
}

/// The components-inverse predicate at the identity natural transformation, per object.
theorem identity_nat_trans_components_inverse_at[O1, M1, O2, M2](f: Functor[O1, M1, O2, M2], x: O1) {
    is_iso_pair(identity_nat_trans(f).src_functor.dst_cat,
        identity_nat_trans(f).component(x),
        identity_nat_trans(f).component(x))
} by {
    let n = identity_nat_trans(f)
    identity_nat_trans_src_functor(f)
    identity_nat_trans_iso_pair_at(f, x)
}

/// The pointwise components of the identity natural transformation are inverses of themselves.
theorem identity_nat_trans_components_inverse[O1, M1, O2, M2](f: Functor[O1, M1, O2, M2]) {
    nat_iso_components_inverse(identity_nat_trans(f), identity_nat_trans(f))
} by {
    let n = identity_nat_trans(f)
    forall(x: O1) {
        identity_nat_trans_components_inverse_at(f, x)
    }
}

/// The identity natural transformation forms a natural isomorphism with itself.
theorem identity_nat_trans_is_natural_isomorphism[O1, M1, O2, M2](f: Functor[O1, M1, O2, M2]) {
    is_natural_isomorphism_pair(identity_nat_trans(f), identity_nat_trans(f))
} by {
    let n = identity_nat_trans(f)
    identity_nat_trans_src_eq_dst(f)
    identity_nat_trans_components_inverse(f)
}

/// Swapping the components of `nat_iso_components_inverse` gives the same predicate
/// using the swapped natural transformation's category.
theorem nat_iso_components_inverse_swap_at[O1, M1, O2, M2](
    forward: NaturalTransformation[O1, M1, O2, M2],
    backward: NaturalTransformation[O1, M1, O2, M2], x: O1
) {
    nat_iso_components_inverse(forward, backward)
        and forward.dst_functor = backward.src_functor
        implies is_iso_pair(backward.src_functor.dst_cat,
            backward.component(x), forward.component(x))
} by {
    if nat_iso_components_inverse(forward, backward)
        and forward.dst_functor = backward.src_functor {
        nat_iso_components_inverse_at(forward, backward, x)
        is_iso_pair_swap(forward.src_functor.dst_cat,
            forward.component(x), backward.component(x))
        nat_trans_shared_dst_cat(forward)
        is_iso_pair(backward.src_functor.dst_cat,
            backward.component(x), forward.component(x))
    }
}

/// Pointwise component-inverse predicate is symmetric for natural isomorphism pairs.
theorem nat_iso_components_inverse_swap[O1, M1, O2, M2](
    forward: NaturalTransformation[O1, M1, O2, M2],
    backward: NaturalTransformation[O1, M1, O2, M2]
) {
    nat_iso_components_inverse(forward, backward)
        and forward.dst_functor = backward.src_functor
        implies nat_iso_components_inverse(backward, forward)
} by {
    if nat_iso_components_inverse(forward, backward)
        and forward.dst_functor = backward.src_functor {
        forall(x: O1) {
            nat_iso_components_inverse_swap_at(forward, backward, x)
            is_iso_pair(backward.src_functor.dst_cat,
                backward.component(x), forward.component(x))
        }
        nat_iso_components_inverse(backward, forward) = forall(x: O1) {
            is_iso_pair(backward.src_functor.dst_cat,
                backward.component(x), forward.component(x))
        }
    }
}

/// Pack the three components of `is_natural_isomorphism_pair`.
theorem is_natural_isomorphism_pair_pack[O1, M1, O2, M2](
    forward: NaturalTransformation[O1, M1, O2, M2],
    backward: NaturalTransformation[O1, M1, O2, M2]
) {
    forward.src_functor = backward.dst_functor
    and forward.dst_functor = backward.src_functor
    and nat_iso_components_inverse(forward, backward)
    implies is_natural_isomorphism_pair(forward, backward)
} by {
    if forward.src_functor = backward.dst_functor
        and forward.dst_functor = backward.src_functor
        and nat_iso_components_inverse(forward, backward) {
        is_natural_isomorphism_pair(forward, backward) =
            (forward.src_functor = backward.dst_functor
            and forward.dst_functor = backward.src_functor
            and nat_iso_components_inverse(forward, backward))
    }
}

/// `is_natural_isomorphism_pair` is symmetric in its two arguments.
theorem is_natural_isomorphism_pair_swap[O1, M1, O2, M2](
    forward: NaturalTransformation[O1, M1, O2, M2],
    backward: NaturalTransformation[O1, M1, O2, M2]
) {
    is_natural_isomorphism_pair(forward, backward) implies
        is_natural_isomorphism_pair(backward, forward)
} by {
    if is_natural_isomorphism_pair(forward, backward) {
        is_natural_isomorphism_pair_src_eq(forward, backward)
        is_natural_isomorphism_pair_dst_eq(forward, backward)
        is_natural_isomorphism_pair_components(forward, backward)
        nat_iso_components_inverse_swap(forward, backward)
        is_natural_isomorphism_pair_pack(backward, forward)
        is_natural_isomorphism_pair(backward, forward)
    }
}

/// At each object, composable pointwise inverse pairs of natural isomorphisms compose to a pointwise inverse pair.
theorem nat_iso_components_inverse_compose_at[O1, M1, O2, M2](
    f1: NaturalTransformation[O1, M1, O2, M2],
    b1: NaturalTransformation[O1, M1, O2, M2],
    f2: NaturalTransformation[O1, M1, O2, M2],
    b2: NaturalTransformation[O1, M1, O2, M2], x: O1
) {
    nat_iso_components_inverse(f1, b1)
        and nat_iso_components_inverse(f2, b2)
        and f1.dst_functor = f2.src_functor
        and f1.src_functor.dst_cat = f2.src_functor.dst_cat
        implies is_iso_pair(f1.src_functor.dst_cat,
            f1.src_functor.dst_cat.compose(f2.component(x), f1.component(x)),
            f1.src_functor.dst_cat.compose(b1.component(x), b2.component(x)))
} by {
    if nat_iso_components_inverse(f1, b1)
        and nat_iso_components_inverse(f2, b2)
        and f1.dst_functor = f2.src_functor
        and f1.src_functor.dst_cat = f2.src_functor.dst_cat {
        let c = f1.src_functor.dst_cat
        // pointwise iso pairs at x
        is_iso_pair(c, f1.component(x), b1.component(x))
        is_iso_pair(f2.src_functor.dst_cat, f2.component(x), b2.component(x))
        // composability of components: c.src(f2.c(x)) = c.dst(f1.c(x))
        nat_trans_component_src(f2, x)
        nat_trans_component_dst(f1, x)
        c.src(f2.component(x)) = c.dst(f1.component(x))
        is_iso_pair_compose(c, f1.component(x), b1.component(x), f2.component(x), b2.component(x))
        is_iso_pair(c, c.compose(f2.component(x), f1.component(x)), c.compose(b1.component(x), b2.component(x)))
        is_iso_pair(f1.src_functor.dst_cat,
            f1.src_functor.dst_cat.compose(f2.component(x), f1.component(x)),
            f1.src_functor.dst_cat.compose(b1.component(x), b2.component(x)))
    }
}

/// Composable natural isomorphism pairs compose pointwise to a natural isomorphism pair.
theorem is_natural_isomorphism_pair_compose[O1, M1, O2, M2](
    f1: NaturalTransformation[O1, M1, O2, M2],
    b1: NaturalTransformation[O1, M1, O2, M2],
    f2: NaturalTransformation[O1, M1, O2, M2],
    b2: NaturalTransformation[O1, M1, O2, M2]
) {
    is_natural_isomorphism_pair(f1, b1)
        and is_natural_isomorphism_pair(f2, b2)
        and f1.dst_functor = f2.src_functor
        implies exists(forward: NaturalTransformation[O1, M1, O2, M2],
                       backward: NaturalTransformation[O1, M1, O2, M2]) {
            vertical_compose_nat_trans(f2, f1) = Option.some(forward)
            and vertical_compose_nat_trans(b1, b2) = Option.some(backward)
            and is_natural_isomorphism_pair(forward, backward)
        }
} by {
    if is_natural_isomorphism_pair(f1, b1)
        and is_natural_isomorphism_pair(f2, b2)
        and f1.dst_functor = f2.src_functor {
        // unpack pair 1
        is_natural_isomorphism_pair(f1, b1) = (f1.src_functor = b1.dst_functor
            and f1.dst_functor = b1.src_functor
            and nat_iso_components_inverse(f1, b1))
        f1.src_functor = b1.dst_functor
        f1.dst_functor = b1.src_functor
        // unpack pair 2
        nat_iso_components_inverse(f2, b2)

        // shared dst_cat
        nat_trans_shared_dst_cat(f1)
        nat_trans_shared_dst_cat(f2)
        f1.src_functor.dst_cat = f2.src_functor.dst_cat
        let c = f1.src_functor.dst_cat

        // forward composability: vertical_composable(f2, f1) requires f2.src_functor = f1.dst_functor
        vertical_compose_nat_trans_some(f2, f1)
        let forward: NaturalTransformation[O1, M1, O2, M2] satisfy {
            vertical_compose_nat_trans(f2, f1) = Option.some(forward)
        }
        vertical_compose_nat_trans_src_functor(f2, f1, forward)
        vertical_compose_nat_trans_dst_functor(f2, f1, forward)
        vertical_compose_nat_trans_component(f2, f1, forward)

        // backward composability: vertical_composable(b1, b2): b1.src_functor = b2.dst_functor
        f2.src_functor = b2.dst_functor
        vertical_composable(b1, b2)
        vertical_compose_nat_trans_some(b1, b2)
        let backward: NaturalTransformation[O1, M1, O2, M2] satisfy {
            vertical_compose_nat_trans(b1, b2) = Option.some(backward)
        }
        vertical_compose_nat_trans_src_functor(b1, b2, backward)
        vertical_compose_nat_trans_dst_functor(b1, b2, backward)
        vertical_compose_nat_trans_component(b1, b2, backward)

        // src/dst alignment
        forward.dst_functor = b2.src_functor
        forward.dst_functor = backward.src_functor

        // pointwise iso pairs of forward and backward
        forall(x: O1) {
            nat_iso_components_inverse_compose_at(f1, b1, f2, b2, x)
            is_iso_pair(c,
                c.compose(f2.component(x), f1.component(x)),
                c.compose(b1.component(x), b2.component(x)))
            // forward.component(x) = vertical_compose_component(f2, f1, x) = c.compose(f2.c(x), f1.c(x))
            vertical_compose_component(f2, f1, x) = c.compose(f2.component(x), f1.component(x))
            nat_trans_shared_dst_cat(b1)
            vertical_compose_component(b1, b2, x) = c.compose(b1.component(x), b2.component(x))
            is_iso_pair(forward.src_functor.dst_cat, forward.component(x), backward.component(x))
        }
        nat_iso_components_inverse(forward, backward)

        is_natural_isomorphism_pair(forward, backward)

        exists(fw: NaturalTransformation[O1, M1, O2, M2], bw: NaturalTransformation[O1, M1, O2, M2]) {
            vertical_compose_nat_trans(f2, f1) = Option.some(fw)
            and vertical_compose_nat_trans(b1, b2) = Option.some(bw)
            and is_natural_isomorphism_pair(fw, bw)
        }
    }
}

/// A natural isomorphism packaged with its forward and backward natural transformations.
structure NaturalIsomorphism[O1, M1, O2, M2] {
    /// The forward natural transformation.
    forward: NaturalTransformation[O1, M1, O2, M2]

    /// The backward natural transformation.
    backward: NaturalTransformation[O1, M1, O2, M2]
} constraint {
    is_natural_isomorphism_pair(forward, backward)
}

/// Construction of a natural isomorphism remembers its forward natural transformation.
theorem nat_iso_new_forward[O1, M1, O2, M2](
    f: NaturalTransformation[O1, M1, O2, M2], b: NaturalTransformation[O1, M1, O2, M2],
    n: NaturalIsomorphism[O1, M1, O2, M2]
) {
    NaturalIsomorphism[O1, M1, O2, M2].new(f, b) = Option.some(n) implies n.forward = f
}

/// Construction of a natural isomorphism remembers its backward natural transformation.
theorem nat_iso_new_backward[O1, M1, O2, M2](
    f: NaturalTransformation[O1, M1, O2, M2], b: NaturalTransformation[O1, M1, O2, M2],
    n: NaturalIsomorphism[O1, M1, O2, M2]
) {
    NaturalIsomorphism[O1, M1, O2, M2].new(f, b) = Option.some(n) implies n.backward = b
}

/// Every natural isomorphism is reconstructed from its components.
theorem nat_iso_new_self[O1, M1, O2, M2](n: NaturalIsomorphism[O1, M1, O2, M2]) {
    NaturalIsomorphism[O1, M1, O2, M2].new(n.forward, n.backward) = Option.some(n)
}

/// Equality of options containing natural isomorphisms is equality of natural isomorphisms.
theorem nat_iso_some_injective[O1, M1, O2, M2](
    n: NaturalIsomorphism[O1, M1, O2, M2], m: NaturalIsomorphism[O1, M1, O2, M2]
) {
    Option.some(n) = Option.some(m) implies n = m
}

/// Natural isomorphisms are equal when their forward and backward components are equal.
theorem nat_iso_ext[O1, M1, O2, M2](
    n: NaturalIsomorphism[O1, M1, O2, M2], m: NaturalIsomorphism[O1, M1, O2, M2]
) {
    n.forward = m.forward and n.backward = m.backward implies n = m
} by {
    if n.forward = m.forward and n.backward = m.backward {
        Option.some(n) = Option.some(m)
        nat_iso_some_injective(n, m)
    }
}

/// The forward and backward of a natural isomorphism form a natural isomorphism pair.
theorem nat_iso_is_pair[O1, M1, O2, M2](n: NaturalIsomorphism[O1, M1, O2, M2]) {
    is_natural_isomorphism_pair(n.forward, n.backward)
} by {
}

/// The source functor of the forward natural transformation matches the target functor of the backward.
theorem nat_iso_src_eq[O1, M1, O2, M2](n: NaturalIsomorphism[O1, M1, O2, M2]) {
    n.forward.src_functor = n.backward.dst_functor
} by {
    nat_iso_is_pair(n)
    is_natural_isomorphism_pair_src_eq(n.forward, n.backward)
}

/// The target functor of the forward natural transformation matches the source functor of the backward.
theorem nat_iso_dst_eq[O1, M1, O2, M2](n: NaturalIsomorphism[O1, M1, O2, M2]) {
    n.forward.dst_functor = n.backward.src_functor
} by {
    nat_iso_is_pair(n)
    is_natural_isomorphism_pair_dst_eq(n.forward, n.backward)
}

/// The components of a natural isomorphism form pointwise inverse pairs.
theorem nat_iso_components[O1, M1, O2, M2](n: NaturalIsomorphism[O1, M1, O2, M2]) {
    nat_iso_components_inverse(n.forward, n.backward)
} by {
    nat_iso_is_pair(n)
    is_natural_isomorphism_pair_components(n.forward, n.backward)
}

/// The identity natural isomorphism on a functor.
let identity_natural_isomorphism[O1, M1, O2, M2](f: Functor[O1, M1, O2, M2]) -> result: NaturalIsomorphism[O1, M1, O2, M2] satisfy {
    result.forward = identity_nat_trans(f) and result.backward = identity_nat_trans(f)
} by {
    identity_nat_trans_is_natural_isomorphism(f)
    let n: NaturalIsomorphism[O1, M1, O2, M2] satisfy {
        NaturalIsomorphism[O1, M1, O2, M2].new(identity_nat_trans(f), identity_nat_trans(f)) = Option.some(n)
    }
    nat_iso_new_forward(identity_nat_trans(f), identity_nat_trans(f), n)
    nat_iso_new_backward(identity_nat_trans(f), identity_nat_trans(f), n)
}

/// The identity natural isomorphism's forward transformation is the identity natural transformation.
theorem identity_natural_isomorphism_forward[O1, M1, O2, M2](f: Functor[O1, M1, O2, M2]) {
    identity_natural_isomorphism(f).forward = identity_nat_trans(f)
}

/// The identity natural isomorphism's backward transformation is the identity natural transformation.
theorem identity_natural_isomorphism_backward[O1, M1, O2, M2](f: Functor[O1, M1, O2, M2]) {
    identity_natural_isomorphism(f).backward = identity_nat_trans(f)
}

/// The inverse natural isomorphism: swap forward and backward.
let inverse_natural_isomorphism[O1, M1, O2, M2](n: NaturalIsomorphism[O1, M1, O2, M2]) -> result: NaturalIsomorphism[O1, M1, O2, M2] satisfy {
    result.forward = n.backward and result.backward = n.forward
} by {
    nat_iso_is_pair(n)
    is_natural_isomorphism_pair_swap(n.forward, n.backward)
    let m: NaturalIsomorphism[O1, M1, O2, M2] satisfy {
        NaturalIsomorphism[O1, M1, O2, M2].new(n.backward, n.forward) = Option.some(m)
    }
    nat_iso_new_forward(n.backward, n.forward, m)
    nat_iso_new_backward(n.backward, n.forward, m)
}

/// The forward of the inverse natural isomorphism is the original backward.
theorem inverse_natural_isomorphism_forward[O1, M1, O2, M2](n: NaturalIsomorphism[O1, M1, O2, M2]) {
    inverse_natural_isomorphism(n).forward = n.backward
}

/// The backward of the inverse natural isomorphism is the original forward.
theorem inverse_natural_isomorphism_backward[O1, M1, O2, M2](n: NaturalIsomorphism[O1, M1, O2, M2]) {
    inverse_natural_isomorphism(n).backward = n.forward
}

/// The source functor of the inverse natural isomorphism's forward transformation is the original forward's target functor.
theorem inverse_natural_isomorphism_forward_src_functor[O1, M1, O2, M2](n: NaturalIsomorphism[O1, M1, O2, M2]) {
    inverse_natural_isomorphism(n).forward.src_functor = n.forward.dst_functor
} by {
    let m = inverse_natural_isomorphism(n)
    inverse_natural_isomorphism_forward(n)
    nat_iso_dst_eq(n)
}

/// The target functor of the inverse natural isomorphism's forward transformation is the original forward's source functor.
theorem inverse_natural_isomorphism_forward_dst_functor[O1, M1, O2, M2](n: NaturalIsomorphism[O1, M1, O2, M2]) {
    inverse_natural_isomorphism(n).forward.dst_functor = n.forward.src_functor
} by {
    let m = inverse_natural_isomorphism(n)
    inverse_natural_isomorphism_forward(n)
    nat_iso_src_eq(n)
}

/// The source functor of the inverse natural isomorphism's backward transformation is the original forward's source functor.
theorem inverse_natural_isomorphism_backward_src_functor[O1, M1, O2, M2](n: NaturalIsomorphism[O1, M1, O2, M2]) {
    inverse_natural_isomorphism(n).backward.src_functor = n.forward.src_functor
} by {
    let m = inverse_natural_isomorphism(n)
    inverse_natural_isomorphism_backward(n)
}

/// The target functor of the inverse natural isomorphism's backward transformation is the original forward's target functor.
theorem inverse_natural_isomorphism_backward_dst_functor[O1, M1, O2, M2](n: NaturalIsomorphism[O1, M1, O2, M2]) {
    inverse_natural_isomorphism(n).backward.dst_functor = n.forward.dst_functor
} by {
    let m = inverse_natural_isomorphism(n)
    inverse_natural_isomorphism_backward(n)
}

/// Taking the inverse twice recovers the original natural isomorphism.
theorem inverse_natural_isomorphism_involutive[O1, M1, O2, M2](n: NaturalIsomorphism[O1, M1, O2, M2]) {
    inverse_natural_isomorphism(inverse_natural_isomorphism(n)) = n
} by {
    let m = inverse_natural_isomorphism(n)
    inverse_natural_isomorphism_forward(n)
    inverse_natural_isomorphism_backward(n)
    let k = inverse_natural_isomorphism(m)
    inverse_natural_isomorphism_forward(m)
    k.forward = n.forward
    inverse_natural_isomorphism_backward(m)
    k.backward = n.backward
    nat_iso_ext(k, n)
}

/// The inverse of the identity natural isomorphism is the identity natural isomorphism.
theorem inverse_identity_natural_isomorphism[O1, M1, O2, M2](f: Functor[O1, M1, O2, M2]) {
    inverse_natural_isomorphism(identity_natural_isomorphism(f)) = identity_natural_isomorphism(f)
} by {
    let i = identity_natural_isomorphism(f)
    identity_natural_isomorphism_forward(f)
    i.forward = identity_nat_trans(f)
    identity_natural_isomorphism_backward(f)
    i.backward = identity_nat_trans(f)
    let j = inverse_natural_isomorphism(i)
    inverse_natural_isomorphism_forward(i)
    j.forward = i.backward
    j.forward = identity_nat_trans(f)
    inverse_natural_isomorphism_backward(i)
    j.backward = i.forward
    j.backward = identity_nat_trans(f)
    nat_iso_ext(j, i)
}

/// At each object, the components of a natural isomorphism form an inverse pair.
theorem nat_iso_at_iso_pair[O1, M1, O2, M2](n: NaturalIsomorphism[O1, M1, O2, M2], x: O1) {
    is_iso_pair(n.forward.src_functor.dst_cat, n.forward.component(x), n.backward.component(x))
} by {
    nat_iso_components(n)
    nat_iso_components_inverse_at(n.forward, n.backward, x)
}

/// The pointwise isomorphism at an object of a natural isomorphism.
let nat_iso_to_iso[O1, M1, O2, M2](n: NaturalIsomorphism[O1, M1, O2, M2], x: O1) -> result: Iso[O2, M2] satisfy {
    Iso[O2, M2].new(n.forward.src_functor.dst_cat, n.forward.component(x), n.backward.component(x)) = Option.some(result)
} by {
    nat_iso_at_iso_pair(n, x)
}

/// The pointwise iso lives in the dst_cat of the forward source functor.
theorem nat_iso_to_iso_cat[O1, M1, O2, M2](n: NaturalIsomorphism[O1, M1, O2, M2], x: O1) {
    nat_iso_to_iso(n, x).cat = n.forward.src_functor.dst_cat
} by {
    let i = nat_iso_to_iso(n, x)
    Iso[O2, M2].new(n.forward.src_functor.dst_cat, n.forward.component(x), n.backward.component(x)) = Option.some(i)
    iso_new_cat(n.forward.src_functor.dst_cat, n.forward.component(x), n.backward.component(x), i)
}

/// The forward morphism of the pointwise iso is the forward natural transformation's component.
theorem nat_iso_to_iso_hom[O1, M1, O2, M2](n: NaturalIsomorphism[O1, M1, O2, M2], x: O1) {
    nat_iso_to_iso(n, x).hom = n.forward.component(x)
} by {
    let i = nat_iso_to_iso(n, x)
    Iso[O2, M2].new(n.forward.src_functor.dst_cat, n.forward.component(x), n.backward.component(x)) = Option.some(i)
    iso_new_hom(n.forward.src_functor.dst_cat, n.forward.component(x), n.backward.component(x), i)
}

/// The inverse morphism of the pointwise iso is the backward natural transformation's component.
theorem nat_iso_to_iso_inv[O1, M1, O2, M2](n: NaturalIsomorphism[O1, M1, O2, M2], x: O1) {
    nat_iso_to_iso(n, x).inv = n.backward.component(x)
} by {
    let i = nat_iso_to_iso(n, x)
    Iso[O2, M2].new(n.forward.src_functor.dst_cat, n.forward.component(x), n.backward.component(x)) = Option.some(i)
    iso_new_inv(n.forward.src_functor.dst_cat, n.forward.component(x), n.backward.component(x), i)
}

/// The source object of the pointwise iso's forward morphism is the forward source functor's image at x.
theorem nat_iso_to_iso_src_hom[O1, M1, O2, M2](n: NaturalIsomorphism[O1, M1, O2, M2], x: O1) {
    nat_iso_to_iso(n, x).cat.src(nat_iso_to_iso(n, x).hom) = n.forward.src_functor.obj_map(x)
} by {
    let i = nat_iso_to_iso(n, x)
    nat_iso_to_iso_cat(n, x)
    nat_iso_to_iso_hom(n, x)
    nat_trans_component_src(n.forward, x)
}

/// The target object of the pointwise iso's forward morphism is the forward target functor's image at x.
theorem nat_iso_to_iso_dst_hom[O1, M1, O2, M2](n: NaturalIsomorphism[O1, M1, O2, M2], x: O1) {
    nat_iso_to_iso(n, x).cat.dst(nat_iso_to_iso(n, x).hom) = n.forward.dst_functor.obj_map(x)
} by {
    let i = nat_iso_to_iso(n, x)
    nat_iso_to_iso_cat(n, x)
    nat_iso_to_iso_hom(n, x)
    nat_trans_component_dst(n.forward, x)
}

/// The source object of the pointwise iso's inverse morphism is the forward target functor's image at x.
theorem nat_iso_to_iso_src_inv[O1, M1, O2, M2](n: NaturalIsomorphism[O1, M1, O2, M2], x: O1) {
    nat_iso_to_iso(n, x).cat.src(nat_iso_to_iso(n, x).inv) = n.forward.dst_functor.obj_map(x)
} by {
    let i = nat_iso_to_iso(n, x)
    iso_src_inv(i)
    nat_iso_to_iso_dst_hom(n, x)
}

/// The target object of the pointwise iso's inverse morphism is the forward source functor's image at x.
theorem nat_iso_to_iso_dst_inv[O1, M1, O2, M2](n: NaturalIsomorphism[O1, M1, O2, M2], x: O1) {
    nat_iso_to_iso(n, x).cat.dst(nat_iso_to_iso(n, x).inv) = n.forward.src_functor.obj_map(x)
} by {
    let i = nat_iso_to_iso(n, x)
    iso_src_hom(i)
    nat_iso_to_iso_src_hom(n, x)
}

/// The pointwise iso of the inverse natural isomorphism equals the inverse iso of the pointwise iso.
theorem inverse_natural_isomorphism_to_iso[O1, M1, O2, M2](n: NaturalIsomorphism[O1, M1, O2, M2], x: O1) {
    nat_iso_to_iso(inverse_natural_isomorphism(n), x) = inverse_iso(nat_iso_to_iso(n, x))
} by {
    let m = inverse_natural_isomorphism(n)
    inverse_natural_isomorphism_forward(n)
    m.forward = n.backward
    inverse_natural_isomorphism_backward(n)
    m.backward = n.forward
    let im = nat_iso_to_iso(m, x)
    nat_iso_to_iso_cat(m, x)
    im.cat = m.forward.src_functor.dst_cat
    im.cat = n.backward.src_functor.dst_cat
    nat_iso_to_iso_hom(m, x)
    im.hom = m.forward.component(x)
    im.hom = n.backward.component(x)
    nat_iso_to_iso_inv(m, x)
    im.inv = m.backward.component(x)
    im.inv = n.forward.component(x)

    let i = nat_iso_to_iso(n, x)
    nat_iso_to_iso_cat(n, x)
    i.cat = n.forward.src_functor.dst_cat
    nat_iso_to_iso_hom(n, x)
    i.hom = n.forward.component(x)
    nat_iso_to_iso_inv(n, x)
    i.inv = n.backward.component(x)

    let j = inverse_iso(i)
    inverse_iso_cat(i)
    j.cat = i.cat
    j.cat = n.forward.src_functor.dst_cat
    inverse_iso_hom(i)
    j.hom = i.inv
    j.hom = n.backward.component(x)
    inverse_iso_inv(i)
    j.inv = i.hom
    j.inv = n.forward.component(x)

    nat_trans_shared_dst_cat(n.forward)
    n.forward.src_functor.dst_cat = n.forward.dst_functor.dst_cat
    nat_iso_dst_eq(n)
    n.forward.dst_functor = n.backward.src_functor
    n.forward.dst_functor.dst_cat = n.backward.src_functor.dst_cat
    n.forward.src_functor.dst_cat = n.backward.src_functor.dst_cat
    im.cat = n.forward.src_functor.dst_cat
    im.cat = j.cat

    iso_ext(im, j)
}

/// The pointwise iso of the identity natural isomorphism is the identity iso.
theorem identity_natural_isomorphism_to_iso[O1, M1, O2, M2](f: Functor[O1, M1, O2, M2], x: O1) {
    nat_iso_to_iso(identity_natural_isomorphism(f), x) = identity_iso(f.dst_cat, f.obj_map(x))
} by {
    let n = identity_natural_isomorphism(f)
    identity_natural_isomorphism_forward(f)
    n.forward = identity_nat_trans(f)
    identity_natural_isomorphism_backward(f)
    n.backward = identity_nat_trans(f)
    identity_nat_trans_src_functor(f)
    n.forward.src_functor = f
    n.forward.src_functor.dst_cat = f.dst_cat

    let i = nat_iso_to_iso(n, x)
    nat_iso_to_iso_cat(n, x)
    i.cat = n.forward.src_functor.dst_cat
    i.cat = f.dst_cat
    nat_iso_to_iso_hom(n, x)
    i.hom = n.forward.component(x)
    identity_nat_trans_component_at(f, x)
    identity_nat_trans(f).component(x) = f.dst_cat.identity(f.obj_map(x))
    n.forward.component(x) = f.dst_cat.identity(f.obj_map(x))
    i.hom = f.dst_cat.identity(f.obj_map(x))
    nat_iso_to_iso_inv(n, x)
    i.inv = n.backward.component(x)
    i.inv = f.dst_cat.identity(f.obj_map(x))

    let j = identity_iso(f.dst_cat, f.obj_map(x))
    identity_iso_cat(f.dst_cat, f.obj_map(x))
    j.cat = f.dst_cat
    identity_iso_hom(f.dst_cat, f.obj_map(x))
    j.hom = f.dst_cat.identity(f.obj_map(x))
    identity_iso_inv(f.dst_cat, f.obj_map(x))
    j.inv = f.dst_cat.identity(f.obj_map(x))

    iso_ext(i, j)
}

/// True if two natural isomorphisms are vertically composable.
define nat_iso_composable[O1, M1, O2, M2](
    n2: NaturalIsomorphism[O1, M1, O2, M2],
    n1: NaturalIsomorphism[O1, M1, O2, M2]
) -> Bool {
    n1.forward.dst_functor = n2.forward.src_functor
}

/// Two composable natural isomorphisms yield forward and backward natural transformations that are an inverse pair.
theorem nat_iso_compose_pair_exists[O1, M1, O2, M2](
    n2: NaturalIsomorphism[O1, M1, O2, M2],
    n1: NaturalIsomorphism[O1, M1, O2, M2]
) {
    nat_iso_composable(n2, n1) implies exists(
        fwd: NaturalTransformation[O1, M1, O2, M2],
        bwd: NaturalTransformation[O1, M1, O2, M2]
    ) {
        vertical_compose_nat_trans(n2.forward, n1.forward) = Option.some(fwd)
        and vertical_compose_nat_trans(n1.backward, n2.backward) = Option.some(bwd)
        and is_natural_isomorphism_pair(fwd, bwd)
    }
} by {
    if nat_iso_composable(n2, n1) {
        nat_iso_is_pair(n1)
        nat_iso_is_pair(n2)
        n1.forward.dst_functor = n2.forward.src_functor
        is_natural_isomorphism_pair_compose(n1.forward, n1.backward, n2.forward, n2.backward)
    }
}

/// Two composable natural isomorphisms yield a packaged natural isomorphism.
theorem nat_iso_compose_exists[O1, M1, O2, M2](
    n2: NaturalIsomorphism[O1, M1, O2, M2],
    n1: NaturalIsomorphism[O1, M1, O2, M2]
) {
    nat_iso_composable(n2, n1) implies exists(
        r: NaturalIsomorphism[O1, M1, O2, M2]
    ) {
        vertical_compose_nat_trans(n2.forward, n1.forward) = Option.some(r.forward)
        and vertical_compose_nat_trans(n1.backward, n2.backward) = Option.some(r.backward)
    }
} by {
    if nat_iso_composable(n2, n1) {
        nat_iso_compose_pair_exists(n2, n1)
        let fwd: NaturalTransformation[O1, M1, O2, M2] satisfy {
            exists(bwd: NaturalTransformation[O1, M1, O2, M2]) {
                vertical_compose_nat_trans(n2.forward, n1.forward) = Option.some(fwd)
                and vertical_compose_nat_trans(n1.backward, n2.backward) = Option.some(bwd)
                and is_natural_isomorphism_pair(fwd, bwd)
            }
        }
        let bwd: NaturalTransformation[O1, M1, O2, M2] satisfy {
            vertical_compose_nat_trans(n2.forward, n1.forward) = Option.some(fwd)
            and vertical_compose_nat_trans(n1.backward, n2.backward) = Option.some(bwd)
            and is_natural_isomorphism_pair(fwd, bwd)
        }
        let r: NaturalIsomorphism[O1, M1, O2, M2] satisfy {
            NaturalIsomorphism[O1, M1, O2, M2].new(fwd, bwd) = Option.some(r)
        }
        nat_iso_new_forward(fwd, bwd, r)
        nat_iso_new_backward(fwd, bwd, r)
        vertical_compose_nat_trans(n1.backward, n2.backward) = Option.some(r.backward)
        exists(s: NaturalIsomorphism[O1, M1, O2, M2]) {
            vertical_compose_nat_trans(n2.forward, n1.forward) = Option.some(s.forward)
            and vertical_compose_nat_trans(n1.backward, n2.backward) = Option.some(s.backward)
        }
    }
}

/// The vertical composition of two natural isomorphisms, defined when they are composable.
let nat_iso_compose[O1, M1, O2, M2](n2: NaturalIsomorphism[O1, M1, O2, M2], n1: NaturalIsomorphism[O1, M1, O2, M2]) -> result: NaturalIsomorphism[O1, M1, O2, M2] satisfy {
    nat_iso_composable(n2, n1) implies (
        vertical_compose_nat_trans(n2.forward, n1.forward) = Option.some(result.forward)
        and vertical_compose_nat_trans(n1.backward, n2.backward) = Option.some(result.backward)
    )
} by {
    if nat_iso_composable(n2, n1) {
        nat_iso_compose_exists(n2, n1)
        let r: NaturalIsomorphism[O1, M1, O2, M2] satisfy {
            vertical_compose_nat_trans(n2.forward, n1.forward) = Option.some(r.forward)
            and vertical_compose_nat_trans(n1.backward, n2.backward) = Option.some(r.backward)
        }
        exists(s: NaturalIsomorphism[O1, M1, O2, M2]) {
            nat_iso_composable(n2, n1) implies (
                vertical_compose_nat_trans(n2.forward, n1.forward) = Option.some(s.forward)
                and vertical_compose_nat_trans(n1.backward, n2.backward) = Option.some(s.backward)
            )
        }
    } else {
        let r = identity_natural_isomorphism(n1.forward.src_functor)
        exists(s: NaturalIsomorphism[O1, M1, O2, M2]) {
            nat_iso_composable(n2, n1) implies (
                vertical_compose_nat_trans(n2.forward, n1.forward) = Option.some(s.forward)
                and vertical_compose_nat_trans(n1.backward, n2.backward) = Option.some(s.backward)
            )
        }
    }
}

/// The forward transformation of a composed natural isomorphism is the vertical composition of the forwards.
theorem nat_iso_compose_forward[O1, M1, O2, M2](
    n2: NaturalIsomorphism[O1, M1, O2, M2],
    n1: NaturalIsomorphism[O1, M1, O2, M2]
) {
    nat_iso_composable(n2, n1) implies
        vertical_compose_nat_trans(n2.forward, n1.forward) = Option.some(nat_iso_compose(n2, n1).forward)
}

/// The backward transformation of a composed natural isomorphism is the reverse vertical composition of the backwards.
theorem nat_iso_compose_backward[O1, M1, O2, M2](
    n2: NaturalIsomorphism[O1, M1, O2, M2],
    n1: NaturalIsomorphism[O1, M1, O2, M2]
) {
    nat_iso_composable(n2, n1) implies
        vertical_compose_nat_trans(n1.backward, n2.backward) = Option.some(nat_iso_compose(n2, n1).backward)
}

/// The source functor of the composed natural isomorphism's forward transformation is the inner forward's source functor.
theorem nat_iso_compose_forward_src_functor[O1, M1, O2, M2](n2: NaturalIsomorphism[O1, M1, O2, M2], n1: NaturalIsomorphism[O1, M1, O2, M2]) {
    nat_iso_composable(n2, n1) implies
        nat_iso_compose(n2, n1).forward.src_functor = n1.forward.src_functor
} by {
    if nat_iso_composable(n2, n1) {
        nat_iso_compose_forward(n2, n1)
        vertical_compose_nat_trans_src_functor(n2.forward, n1.forward, nat_iso_compose(n2, n1).forward)
    }
}

/// The target functor of the composed natural isomorphism's forward transformation is the outer forward's target functor.
theorem nat_iso_compose_forward_dst_functor[O1, M1, O2, M2](n2: NaturalIsomorphism[O1, M1, O2, M2], n1: NaturalIsomorphism[O1, M1, O2, M2]) {
    nat_iso_composable(n2, n1) implies
        nat_iso_compose(n2, n1).forward.dst_functor = n2.forward.dst_functor
} by {
    if nat_iso_composable(n2, n1) {
        nat_iso_compose_forward(n2, n1)
        vertical_compose_nat_trans_dst_functor(n2.forward, n1.forward, nat_iso_compose(n2, n1).forward)
    }
}

/// The source functor of the composed natural isomorphism's backward transformation is the outer backward's source functor.
theorem nat_iso_compose_backward_src_functor[O1, M1, O2, M2](n2: NaturalIsomorphism[O1, M1, O2, M2], n1: NaturalIsomorphism[O1, M1, O2, M2]) {
    nat_iso_composable(n2, n1) implies
        nat_iso_compose(n2, n1).backward.src_functor = n2.backward.src_functor
} by {
    if nat_iso_composable(n2, n1) {
        nat_iso_compose_backward(n2, n1)
        vertical_compose_nat_trans_src_functor(n1.backward, n2.backward, nat_iso_compose(n2, n1).backward)
    }
}

/// The target functor of the composed natural isomorphism's backward transformation is the inner backward's target functor.
theorem nat_iso_compose_backward_dst_functor[O1, M1, O2, M2](n2: NaturalIsomorphism[O1, M1, O2, M2], n1: NaturalIsomorphism[O1, M1, O2, M2]) {
    nat_iso_composable(n2, n1) implies
        nat_iso_compose(n2, n1).backward.dst_functor = n1.backward.dst_functor
} by {
    if nat_iso_composable(n2, n1) {
        nat_iso_compose_backward(n2, n1)
        vertical_compose_nat_trans_dst_functor(n1.backward, n2.backward, nat_iso_compose(n2, n1).backward)
    }
}
