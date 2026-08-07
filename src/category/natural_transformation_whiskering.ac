/// Whiskering API for natural transformations.

from category.category import category_id_right
from category.functor import Functor, functor_src, functor_dst, functor_identity, functor_compose, compose_functor
from category.natural_transformation import NaturalTransformation, nat_trans_naturality,
    vertical_composable, vertical_compose_component, vertical_compose_components_composable,
    vertical_compose_nat_trans, vertical_compose_nat_trans_component,
    nat_trans_component_src, nat_trans_component_dst, identity_nat_trans, identity_nat_trans_src_functor,
    identity_nat_trans_dst_functor, identity_nat_trans_component,
    identity_nat_trans_component_eq, horizontal_composable,
    horizontal_compose_component, horizontal_compose_nat_trans,
    horizontal_compose_nat_trans_some, horizontal_compose_nat_trans_src_functor,
    horizontal_compose_nat_trans_dst_functor, horizontal_compose_nat_trans_component,
    horizontal_compose_identity_outer_nat_trans_component

/// True if a functor can be used to left-whisker a natural transformation.
define left_whisker_composable[O1, M1, O2, M2, O3, M3](
    h: Functor[O1, M1, O2, M2],
    a: NaturalTransformation[O2, M2, O3, M3]
) -> Bool {
    a.src_functor.src_cat = h.dst_cat
}

/// The left-whiskered component at an object.
define left_whisker_nat_trans_component[O1, M1, O2, M2, O3, M3](
    h: Functor[O1, M1, O2, M2],
    a: NaturalTransformation[O2, M2, O3, M3],
    x: O1
) -> M3 {
    a.component(h.obj_map(x))
}

/// The left-whiskered component is the original component at the functor image.
theorem left_whisker_nat_trans_component_eq[O1, M1, O2, M2, O3, M3](
    h: Functor[O1, M1, O2, M2],
    a: NaturalTransformation[O2, M2, O3, M3],
    x: O1
) {
    left_whisker_nat_trans_component(h, a, x) = a.component(h.obj_map(x))
}

/// The bundled left whiskering of a natural transformation by a functor.
define left_whisker_nat_trans[O1, M1, O2, M2, O3, M3](
    h: Functor[O1, M1, O2, M2],
    a: NaturalTransformation[O2, M2, O3, M3],
    src_comp: Functor[O1, M1, O3, M3],
    dst_comp: Functor[O1, M1, O3, M3]
) -> Option[NaturalTransformation[O1, M1, O3, M3]] {
    horizontal_compose_nat_trans(a, identity_nat_trans(h), src_comp, dst_comp)
}

/// Left whiskering exists when the input functor and transformation are composable and the two composite functors are chosen.
theorem left_whisker_nat_trans_some[O1, M1, O2, M2, O3, M3](
    h: Functor[O1, M1, O2, M2],
    a: NaturalTransformation[O2, M2, O3, M3],
    src_comp: Functor[O1, M1, O3, M3],
    dst_comp: Functor[O1, M1, O3, M3]
) {
    left_whisker_composable(h, a) and
    compose_functor(a.src_functor, h) = Option.some(src_comp) and
    compose_functor(a.dst_functor, h) = Option.some(dst_comp) implies
    exists(n: NaturalTransformation[O1, M1, O3, M3]) {
        left_whisker_nat_trans(h, a, src_comp, dst_comp) = Option.some(n)
    }
} by {
    let id_h = identity_nat_trans(h)
    if left_whisker_composable(h, a) and
        compose_functor(a.src_functor, h) = Option.some(src_comp) and
        compose_functor(a.dst_functor, h) = Option.some(dst_comp) {
        identity_nat_trans_src_functor(h)
        identity_nat_trans_dst_functor(h)
        compose_functor(a.src_functor, id_h.src_functor) = Option.some(src_comp)
        compose_functor(a.dst_functor, id_h.dst_functor) = Option.some(dst_comp)
        horizontal_compose_nat_trans_some(a, id_h, src_comp, dst_comp)
        let n: NaturalTransformation[O1, M1, O3, M3] satisfy {
            horizontal_compose_nat_trans(a, id_h, src_comp, dst_comp) = Option.some(n)
        }
        left_whisker_nat_trans(h, a, src_comp, dst_comp) = Option.some(n)
    }
}

/// A bundled left whiskering has the chosen source composite as source functor.
theorem left_whisker_nat_trans_src_functor[O1, M1, O2, M2, O3, M3](
    h: Functor[O1, M1, O2, M2],
    a: NaturalTransformation[O2, M2, O3, M3],
    src_comp: Functor[O1, M1, O3, M3],
    dst_comp: Functor[O1, M1, O3, M3],
    n: NaturalTransformation[O1, M1, O3, M3]
) {
    left_whisker_nat_trans(h, a, src_comp, dst_comp) = Option.some(n) implies n.src_functor = src_comp
} by {
    let id_h = identity_nat_trans(h)
    if left_whisker_nat_trans(h, a, src_comp, dst_comp) = Option.some(n) {
        horizontal_compose_nat_trans(a, id_h, src_comp, dst_comp) = Option.some(n)
        horizontal_compose_nat_trans_src_functor(a, id_h, src_comp, dst_comp, n)
    }
}

/// A bundled left whiskering has the chosen target composite as target functor.
theorem left_whisker_nat_trans_dst_functor[O1, M1, O2, M2, O3, M3](
    h: Functor[O1, M1, O2, M2],
    a: NaturalTransformation[O2, M2, O3, M3],
    src_comp: Functor[O1, M1, O3, M3],
    dst_comp: Functor[O1, M1, O3, M3],
    n: NaturalTransformation[O1, M1, O3, M3]
) {
    left_whisker_nat_trans(h, a, src_comp, dst_comp) = Option.some(n) implies n.dst_functor = dst_comp
} by {
    let id_h = identity_nat_trans(h)
    if left_whisker_nat_trans(h, a, src_comp, dst_comp) = Option.some(n) {
        horizontal_compose_nat_trans(a, id_h, src_comp, dst_comp) = Option.some(n)
        horizontal_compose_nat_trans_dst_functor(a, id_h, src_comp, dst_comp, n)
    }
}

/// The component map of a bundled left whiskering is the standard horizontal component map.
theorem left_whisker_nat_trans_component_map[O1, M1, O2, M2, O3, M3](
    h: Functor[O1, M1, O2, M2],
    a: NaturalTransformation[O2, M2, O3, M3],
    src_comp: Functor[O1, M1, O3, M3],
    dst_comp: Functor[O1, M1, O3, M3],
    n: NaturalTransformation[O1, M1, O3, M3]
) {
    left_whisker_nat_trans(h, a, src_comp, dst_comp) = Option.some(n) implies
    n.component = horizontal_compose_component(a, identity_nat_trans(h))
} by {
    let id_h = identity_nat_trans(h)
    if left_whisker_nat_trans(h, a, src_comp, dst_comp) = Option.some(n) {
        horizontal_compose_nat_trans(a, id_h, src_comp, dst_comp) = Option.some(n)
        horizontal_compose_nat_trans_component(a, id_h, src_comp, dst_comp, n)
    }
}

/// Left-whiskered components have the expected source endpoint.
theorem left_whisker_nat_trans_component_src[O1, M1, O2, M2, O3, M3](
    h: Functor[O1, M1, O2, M2],
    a: NaturalTransformation[O2, M2, O3, M3],
    x: O1
) {
    left_whisker_composable(h, a) implies
    a.src_functor.dst_cat.src(left_whisker_nat_trans_component(h, a, x)) =
        a.src_functor.obj_map(h.obj_map(x))
} by {
    if left_whisker_composable(h, a) {
        left_whisker_nat_trans_component(h, a, x) = a.component(h.obj_map(x))
        nat_trans_component_src(a, h.obj_map(x))
    }
}

/// Left-whiskered components have the expected target endpoint.
theorem left_whisker_nat_trans_component_dst[O1, M1, O2, M2, O3, M3](
    h: Functor[O1, M1, O2, M2],
    a: NaturalTransformation[O2, M2, O3, M3],
    x: O1
) {
    left_whisker_composable(h, a) implies
    a.src_functor.dst_cat.dst(left_whisker_nat_trans_component(h, a, x)) =
        a.dst_functor.obj_map(h.obj_map(x))
} by {
    if left_whisker_composable(h, a) {
        left_whisker_nat_trans_component(h, a, x) = a.component(h.obj_map(x))
        nat_trans_component_dst(a, h.obj_map(x))
    }
}

/// Left-whiskering has the expected component formula at each object.
theorem left_whisker_nat_trans_component_at[O1, M1, O2, M2, O3, M3](
    h: Functor[O1, M1, O2, M2],
    a: NaturalTransformation[O2, M2, O3, M3],
    src_comp: Functor[O1, M1, O3, M3],
    dst_comp: Functor[O1, M1, O3, M3],
    n: NaturalTransformation[O1, M1, O3, M3],
    x: O1
) {
    left_whisker_composable(h, a) and
    left_whisker_nat_trans(h, a, src_comp, dst_comp) = Option.some(n) implies
    n.component(x) = left_whisker_nat_trans_component(h, a, x)
} by {
    let id_h = identity_nat_trans(h)
    if left_whisker_composable(h, a) and
        left_whisker_nat_trans(h, a, src_comp, dst_comp) = Option.some(n) {
        left_whisker_nat_trans_component_map(h, a, src_comp, dst_comp, n)
        n.component = horizontal_compose_component(a, id_h)
        n.component(x) = horizontal_compose_component(a, id_h)(x)
        horizontal_compose_component(a, id_h)(x) = horizontal_compose_component(a, id_h, x)
        identity_nat_trans_dst_functor(h)
        identity_nat_trans_component_eq(h)
        identity_nat_trans_component(h)(x) = identity_nat_trans_component(h, x)
        h.dst_cat.identity(h.obj_map(x)) = a.src_functor.src_cat.identity(h.obj_map(x))
        functor_identity(a.src_functor, h.obj_map(x))
        nat_trans_component_src(a, h.obj_map(x))
        category_id_right(a.src_functor.dst_cat, a.component(h.obj_map(x)))
        horizontal_compose_component(a, id_h, x) = a.component(h.obj_map(x))
        n.component(x) = left_whisker_nat_trans_component(h, a, x)
    }
}

/// Naturality of a bundled left whiskering, pointwise.
theorem left_whisker_nat_trans_naturality_at[O1, M1, O2, M2, O3, M3](
    h: Functor[O1, M1, O2, M2],
    a: NaturalTransformation[O2, M2, O3, M3],
    src_comp: Functor[O1, M1, O3, M3],
    dst_comp: Functor[O1, M1, O3, M3],
    n: NaturalTransformation[O1, M1, O3, M3],
    m: M1
) {
    left_whisker_nat_trans(h, a, src_comp, dst_comp) = Option.some(n) implies
    n.src_functor.dst_cat.compose(n.component(n.src_functor.src_cat.dst(m)), n.src_functor.mor_map(m)) =
        n.src_functor.dst_cat.compose(n.dst_functor.mor_map(m), n.component(n.src_functor.src_cat.src(m)))
} by {
    if left_whisker_nat_trans(h, a, src_comp, dst_comp) = Option.some(n) {
        nat_trans_naturality(n, m)
    }
}

/// True if a functor can be used to right-whisker a natural transformation.
define right_whisker_composable[O1, M1, O2, M2, O3, M3](
    a: NaturalTransformation[O1, M1, O2, M2],
    k: Functor[O2, M2, O3, M3]
) -> Bool {
    k.src_cat = a.src_functor.dst_cat
}

/// The right-whiskered component at an object.
define right_whisker_nat_trans_component[O1, M1, O2, M2, O3, M3](
    a: NaturalTransformation[O1, M1, O2, M2],
    k: Functor[O2, M2, O3, M3],
    x: O1
) -> M3 {
    k.mor_map(a.component(x))
}

/// The right-whiskered component is the functor image of the original component.
theorem right_whisker_nat_trans_component_eq[O1, M1, O2, M2, O3, M3](
    a: NaturalTransformation[O1, M1, O2, M2],
    k: Functor[O2, M2, O3, M3],
    x: O1
) {
    right_whisker_nat_trans_component(a, k, x) = k.mor_map(a.component(x))
}

/// The bundled right whiskering of a natural transformation by a functor.
define right_whisker_nat_trans[O1, M1, O2, M2, O3, M3](
    a: NaturalTransformation[O1, M1, O2, M2],
    k: Functor[O2, M2, O3, M3],
    src_comp: Functor[O1, M1, O3, M3],
    dst_comp: Functor[O1, M1, O3, M3]
) -> Option[NaturalTransformation[O1, M1, O3, M3]] {
    horizontal_compose_nat_trans(identity_nat_trans(k), a, src_comp, dst_comp)
}

/// Right whiskering exists when the input transformation and functor are composable and the two composite functors are chosen.
theorem right_whisker_nat_trans_some[O1, M1, O2, M2, O3, M3](
    a: NaturalTransformation[O1, M1, O2, M2],
    k: Functor[O2, M2, O3, M3],
    src_comp: Functor[O1, M1, O3, M3],
    dst_comp: Functor[O1, M1, O3, M3]
) {
    right_whisker_composable(a, k) and
    compose_functor(k, a.src_functor) = Option.some(src_comp) and
    compose_functor(k, a.dst_functor) = Option.some(dst_comp) implies
    exists(n: NaturalTransformation[O1, M1, O3, M3]) {
        right_whisker_nat_trans(a, k, src_comp, dst_comp) = Option.some(n)
    }
} by {
    let id_k = identity_nat_trans(k)
    if right_whisker_composable(a, k) and
        compose_functor(k, a.src_functor) = Option.some(src_comp) and
        compose_functor(k, a.dst_functor) = Option.some(dst_comp) {
        identity_nat_trans_src_functor(k)
        identity_nat_trans_dst_functor(k)
        compose_functor(id_k.src_functor, a.src_functor) = Option.some(src_comp)
        compose_functor(id_k.dst_functor, a.dst_functor) = Option.some(dst_comp)
        horizontal_compose_nat_trans_some(id_k, a, src_comp, dst_comp)
        let n: NaturalTransformation[O1, M1, O3, M3] satisfy {
            horizontal_compose_nat_trans(id_k, a, src_comp, dst_comp) = Option.some(n)
        }
        right_whisker_nat_trans(a, k, src_comp, dst_comp) = Option.some(n)
    }
}

/// A bundled right whiskering has the chosen source composite as source functor.
theorem right_whisker_nat_trans_src_functor[O1, M1, O2, M2, O3, M3](
    a: NaturalTransformation[O1, M1, O2, M2],
    k: Functor[O2, M2, O3, M3],
    src_comp: Functor[O1, M1, O3, M3],
    dst_comp: Functor[O1, M1, O3, M3],
    n: NaturalTransformation[O1, M1, O3, M3]
) {
    right_whisker_nat_trans(a, k, src_comp, dst_comp) = Option.some(n) implies n.src_functor = src_comp
} by {
    let id_k = identity_nat_trans(k)
    if right_whisker_nat_trans(a, k, src_comp, dst_comp) = Option.some(n) {
        horizontal_compose_nat_trans(id_k, a, src_comp, dst_comp) = Option.some(n)
        horizontal_compose_nat_trans_src_functor(id_k, a, src_comp, dst_comp, n)
    }
}

/// A bundled right whiskering has the chosen target composite as target functor.
theorem right_whisker_nat_trans_dst_functor[O1, M1, O2, M2, O3, M3](
    a: NaturalTransformation[O1, M1, O2, M2],
    k: Functor[O2, M2, O3, M3],
    src_comp: Functor[O1, M1, O3, M3],
    dst_comp: Functor[O1, M1, O3, M3],
    n: NaturalTransformation[O1, M1, O3, M3]
) {
    right_whisker_nat_trans(a, k, src_comp, dst_comp) = Option.some(n) implies n.dst_functor = dst_comp
} by {
    let id_k = identity_nat_trans(k)
    if right_whisker_nat_trans(a, k, src_comp, dst_comp) = Option.some(n) {
        horizontal_compose_nat_trans(id_k, a, src_comp, dst_comp) = Option.some(n)
        horizontal_compose_nat_trans_dst_functor(id_k, a, src_comp, dst_comp, n)
    }
}

/// The component map of a bundled right whiskering is the standard right-whiskered component map.
theorem right_whisker_nat_trans_component_map[O1, M1, O2, M2, O3, M3](
    a: NaturalTransformation[O1, M1, O2, M2],
    k: Functor[O2, M2, O3, M3],
    src_comp: Functor[O1, M1, O3, M3],
    dst_comp: Functor[O1, M1, O3, M3],
    n: NaturalTransformation[O1, M1, O3, M3]
) {
    right_whisker_composable(a, k) and
    compose_functor(k, a.src_functor) = Option.some(src_comp) and
    compose_functor(k, a.dst_functor) = Option.some(dst_comp) and
    right_whisker_nat_trans(a, k, src_comp, dst_comp) = Option.some(n) implies
    n.component = right_whisker_nat_trans_component(a, k)
} by {
    let id_k = identity_nat_trans(k)
    if right_whisker_composable(a, k) and
        compose_functor(k, a.src_functor) = Option.some(src_comp) and
        compose_functor(k, a.dst_functor) = Option.some(dst_comp) and
        right_whisker_nat_trans(a, k, src_comp, dst_comp) = Option.some(n) {
        right_whisker_nat_trans(a, k, src_comp, dst_comp) =
            horizontal_compose_nat_trans(id_k, a, src_comp, dst_comp)
        horizontal_compose_nat_trans(id_k, a, src_comp, dst_comp) = Option.some(n)
        horizontal_composable(identity_nat_trans(k), a)
        horizontal_compose_identity_outer_nat_trans_component(k, a, src_comp, dst_comp, n)
        forall(x: O1) {
            right_whisker_nat_trans_component(a, k)(x) = right_whisker_nat_trans_component(a, k, x)
            (function(y: O1) { k.mor_map(a.component(y)) })(x) = k.mor_map(a.component(x))
            n.component(x) = right_whisker_nat_trans_component(a, k)(x)
        }
        n.component = right_whisker_nat_trans_component(a, k)
    }
}

/// Right-whiskered components have the expected source endpoint.
theorem right_whisker_nat_trans_component_src[O1, M1, O2, M2, O3, M3](
    a: NaturalTransformation[O1, M1, O2, M2],
    k: Functor[O2, M2, O3, M3],
    x: O1
) {
    right_whisker_composable(a, k) implies
    k.dst_cat.src(right_whisker_nat_trans_component(a, k, x)) =
        k.obj_map(a.src_functor.obj_map(x))
} by {
    if right_whisker_composable(a, k) {
        functor_src(k, a.component(x))
        nat_trans_component_src(a, x)
        k.src_cat.src = a.src_functor.dst_cat.src
        k.dst_cat.src(right_whisker_nat_trans_component(a, k, x)) = k.obj_map(a.src_functor.obj_map(x))
    }
}

/// Right-whiskered components have the expected target endpoint.
theorem right_whisker_nat_trans_component_dst[O1, M1, O2, M2, O3, M3](
    a: NaturalTransformation[O1, M1, O2, M2],
    k: Functor[O2, M2, O3, M3],
    x: O1
) {
    right_whisker_composable(a, k) implies
    k.dst_cat.dst(right_whisker_nat_trans_component(a, k, x)) =
        k.obj_map(a.dst_functor.obj_map(x))
} by {
    if right_whisker_composable(a, k) {
        functor_dst(k, a.component(x))
        nat_trans_component_dst(a, x)
        k.src_cat.dst = a.src_functor.dst_cat.dst
        k.dst_cat.dst(right_whisker_nat_trans_component(a, k, x)) = k.obj_map(a.dst_functor.obj_map(x))
    }
}

/// Right-whiskering has the expected component formula at each object.
theorem right_whisker_nat_trans_component_at[O1, M1, O2, M2, O3, M3](
    a: NaturalTransformation[O1, M1, O2, M2],
    k: Functor[O2, M2, O3, M3],
    src_comp: Functor[O1, M1, O3, M3],
    dst_comp: Functor[O1, M1, O3, M3],
    n: NaturalTransformation[O1, M1, O3, M3],
    x: O1
) {
    right_whisker_composable(a, k) and
    compose_functor(k, a.src_functor) = Option.some(src_comp) and
    compose_functor(k, a.dst_functor) = Option.some(dst_comp) and
    right_whisker_nat_trans(a, k, src_comp, dst_comp) = Option.some(n) implies
    n.component(x) = right_whisker_nat_trans_component(a, k, x)
} by {
    if right_whisker_composable(a, k) and
        compose_functor(k, a.src_functor) = Option.some(src_comp) and
        compose_functor(k, a.dst_functor) = Option.some(dst_comp) and
        right_whisker_nat_trans(a, k, src_comp, dst_comp) = Option.some(n) {
        right_whisker_nat_trans_component_map(a, k, src_comp, dst_comp, n)
        right_whisker_nat_trans_component(a, k)(x) = right_whisker_nat_trans_component(a, k, x)
        n.component(x) = right_whisker_nat_trans_component(a, k, x)
    }
}

/// Naturality of a bundled right whiskering, pointwise.
theorem right_whisker_nat_trans_naturality_at[O1, M1, O2, M2, O3, M3](
    a: NaturalTransformation[O1, M1, O2, M2],
    k: Functor[O2, M2, O3, M3],
    src_comp: Functor[O1, M1, O3, M3],
    dst_comp: Functor[O1, M1, O3, M3],
    n: NaturalTransformation[O1, M1, O3, M3],
    m: M1
) {
    right_whisker_nat_trans(a, k, src_comp, dst_comp) = Option.some(n) implies
    n.src_functor.dst_cat.compose(n.component(n.src_functor.src_cat.dst(m)), n.src_functor.mor_map(m)) =
        n.src_functor.dst_cat.compose(n.dst_functor.mor_map(m), n.component(n.src_functor.src_cat.src(m)))
} by {
    if right_whisker_nat_trans(a, k, src_comp, dst_comp) = Option.some(n) {
        nat_trans_naturality(n, m)
    }
}

/// Left whiskering sends the component of a vertical composite to the pointwise composite of the left-whiskered components.
theorem left_whisker_vertical_compose_component_at[O1, M1, O2, M2, O3, M3](
    h: Functor[O1, M1, O2, M2],
    upper: NaturalTransformation[O2, M2, O3, M3],
    lower: NaturalTransformation[O2, M2, O3, M3],
    composite: NaturalTransformation[O2, M2, O3, M3],
    x: O1
) {
    vertical_compose_nat_trans(upper, lower) = Option.some(composite) implies
    left_whisker_nat_trans_component(h, composite, x) =
        upper.src_functor.dst_cat.compose(
            left_whisker_nat_trans_component(h, upper, x),
            left_whisker_nat_trans_component(h, lower, x))
} by {
    if vertical_compose_nat_trans(upper, lower) = Option.some(composite) {
        vertical_compose_nat_trans_component(upper, lower, composite)
        composite.component = vertical_compose_component(upper, lower)
        left_whisker_nat_trans_component(h, composite, x) = composite.component(h.obj_map(x))
        composite.component(h.obj_map(x)) = vertical_compose_component(upper, lower)(h.obj_map(x))
        vertical_compose_component(upper, lower)(h.obj_map(x)) =
            vertical_compose_component(upper, lower, h.obj_map(x))
        vertical_compose_component(upper, lower, h.obj_map(x)) =
            upper.src_functor.dst_cat.compose(upper.component(h.obj_map(x)), lower.component(h.obj_map(x)))
        left_whisker_nat_trans_component(h, upper, x) = upper.component(h.obj_map(x))
        left_whisker_nat_trans_component(h, lower, x) = lower.component(h.obj_map(x))
        left_whisker_nat_trans_component(h, composite, x) =
            upper.src_functor.dst_cat.compose(
                left_whisker_nat_trans_component(h, upper, x),
                left_whisker_nat_trans_component(h, lower, x))
    }
}

/// Right whiskering sends the component of a vertical composite to the pointwise composite of the right-whiskered components.
theorem right_whisker_vertical_compose_component_at[O1, M1, O2, M2, O3, M3](
    upper: NaturalTransformation[O1, M1, O2, M2],
    lower: NaturalTransformation[O1, M1, O2, M2],
    composite: NaturalTransformation[O1, M1, O2, M2],
    k: Functor[O2, M2, O3, M3],
    x: O1
) {
    vertical_composable(upper, lower) and
    right_whisker_composable(upper, k) and
    vertical_compose_nat_trans(upper, lower) = Option.some(composite) implies
    right_whisker_nat_trans_component(composite, k, x) =
        k.dst_cat.compose(
            right_whisker_nat_trans_component(upper, k, x),
            right_whisker_nat_trans_component(lower, k, x))
} by {
    if vertical_composable(upper, lower) and
        right_whisker_composable(upper, k) and
        vertical_compose_nat_trans(upper, lower) = Option.some(composite) {
        vertical_compose_nat_trans_component(upper, lower, composite)
        composite.component = vertical_compose_component(upper, lower)
        right_whisker_nat_trans_component(composite, k, x) = k.mor_map(composite.component(x))
        composite.component(x) = vertical_compose_component(upper, lower)(x)
        vertical_compose_component(upper, lower)(x) = vertical_compose_component(upper, lower, x)
        vertical_compose_component(upper, lower, x) =
            upper.src_functor.dst_cat.compose(upper.component(x), lower.component(x))
        right_whisker_composable(upper, k) = (k.src_cat = upper.src_functor.dst_cat)
        k.src_cat = upper.src_functor.dst_cat
        vertical_compose_components_composable(upper, lower, x)
        upper.src_functor.dst_cat.src(upper.component(x)) =
            upper.src_functor.dst_cat.dst(lower.component(x))
        k.src_cat.src = upper.src_functor.dst_cat.src
        k.src_cat.dst = upper.src_functor.dst_cat.dst
        k.src_cat.compose = upper.src_functor.dst_cat.compose
        k.src_cat.src(upper.component(x)) = k.src_cat.dst(lower.component(x))
        functor_compose(k, upper.component(x), lower.component(x))
        k.mor_map(k.src_cat.compose(upper.component(x), lower.component(x))) =
            k.dst_cat.compose(k.mor_map(upper.component(x)), k.mor_map(lower.component(x)))
        k.src_cat.compose(upper.component(x), lower.component(x)) =
            upper.src_functor.dst_cat.compose(upper.component(x), lower.component(x))
        k.mor_map(vertical_compose_component(upper, lower, x)) =
            k.dst_cat.compose(k.mor_map(upper.component(x)), k.mor_map(lower.component(x)))
        k.mor_map(composite.component(x)) =
            k.dst_cat.compose(k.mor_map(upper.component(x)), k.mor_map(lower.component(x)))
        right_whisker_nat_trans_component(upper, k, x) = k.mor_map(upper.component(x))
        right_whisker_nat_trans_component(lower, k, x) = k.mor_map(lower.component(x))
        right_whisker_nat_trans_component(composite, k, x) =
            k.dst_cat.compose(
                right_whisker_nat_trans_component(upper, k, x),
                right_whisker_nat_trans_component(lower, k, x))
    }
}

/// Horizontal composition components are the composite of the left- and right-whiskered components.
theorem horizontal_compose_component_via_whiskering[O1, M1, O2, M2, O3, M3](
    outer: NaturalTransformation[O2, M2, O3, M3],
    inner: NaturalTransformation[O1, M1, O2, M2],
    x: O1
) {
    horizontal_compose_component(outer, inner, x) = outer.src_functor.dst_cat.compose(
        left_whisker_nat_trans_component(inner.dst_functor, outer, x),
        right_whisker_nat_trans_component(inner, outer.src_functor, x))
} by {
    left_whisker_nat_trans_component(inner.dst_functor, outer, x) =
        outer.component(inner.dst_functor.obj_map(x))
    right_whisker_nat_trans_component(inner, outer.src_functor, x) =
        outer.src_functor.mor_map(inner.component(x))
}

/// A bundled horizontal composition has the same component formula via whiskering.
theorem horizontal_compose_nat_trans_component_via_whiskering[O1, M1, O2, M2, O3, M3](
    outer: NaturalTransformation[O2, M2, O3, M3],
    inner: NaturalTransformation[O1, M1, O2, M2],
    src_comp: Functor[O1, M1, O3, M3],
    dst_comp: Functor[O1, M1, O3, M3],
    n: NaturalTransformation[O1, M1, O3, M3],
    x: O1
) {
    horizontal_compose_nat_trans(outer, inner, src_comp, dst_comp) = Option.some(n) implies
    n.component(x) = outer.src_functor.dst_cat.compose(
        left_whisker_nat_trans_component(inner.dst_functor, outer, x),
        right_whisker_nat_trans_component(inner, outer.src_functor, x))
} by {
    if horizontal_compose_nat_trans(outer, inner, src_comp, dst_comp) = Option.some(n) {
        horizontal_compose_nat_trans_component(outer, inner, src_comp, dst_comp, n)
        horizontal_compose_component(outer, inner)(x) = horizontal_compose_component(outer, inner, x)
        horizontal_compose_component_via_whiskering(outer, inner, x)
        n.component(x) = outer.src_functor.dst_cat.compose(
            left_whisker_nat_trans_component(inner.dst_functor, outer, x),
            right_whisker_nat_trans_component(inner, outer.src_functor, x))
    }
}
