from int import Int, abs, abs_from_nat
from algebra.add_comm_monoid_rearrange import add_swap_inner
from nat import Nat
from pair import Pair
from zmod import int_mod_rel
from data.int.int_congruence import int_congr_mod_iff_int_mod_rel, int_congr_mod_zero_iff_divides,
    int_congr_mod_add_right
from number_theory import congruence_class_contains, nat_bezout
from residue_class_disjoint import classes_disjoint, classes_disjoint_apply,
    congruence_class_contains_eq
from residue_class_lcm import classes_disjoint_of_gcd_not_divides

numerals Int

/// A modulus dividing the gap to the residue puts an integer in the class.
///
/// The converse of `class_modulus_divides_difference`, obtained by running the same chain of
/// congruence rewrites backwards. Producing a member of a class needs this direction.
theorem congruence_class_contains_of_divides(cl: Pair[Nat, Nat], x: Int) {
    Int.from_nat(cl.first).divides(x - Int.from_nat(cl.second))
        implies congruence_class_contains(cl, x)
} by {
    if Int.from_nat(cl.first).divides(x - Int.from_nat(cl.second)) {
        int_congr_mod_zero_iff_divides(x - Int.from_nat(cl.second), cl.first)
        ((x - Int.from_nat(cl.second)).congr_mod(Int.0, cl.first)
            = Int.from_nat(cl.first).divides(x - Int.from_nat(cl.second)))
        (x - Int.from_nat(cl.second)).congr_mod(Int.0, cl.first)
        int_congr_mod_add_right(x - Int.from_nat(cl.second), Int.0,
            Int.from_nat(cl.second), cl.first)
        ((x - Int.from_nat(cl.second) + Int.from_nat(cl.second)).congr_mod(
            Int.0 + Int.from_nat(cl.second), cl.first))
        (x - Int.from_nat(cl.second) + Int.from_nat(cl.second) = x)
        (Int.0 + Int.from_nat(cl.second) = Int.from_nat(cl.second))
        x.congr_mod(Int.from_nat(cl.second), cl.first)
        int_congr_mod_iff_int_mod_rel(x, Int.from_nat(cl.second), cl.first)
        (x.congr_mod(Int.from_nat(cl.second), cl.first)
            = int_mod_rel(cl.first, x, Int.from_nat(cl.second)))
        int_mod_rel(cl.first, x, Int.from_nat(cl.second))
        congruence_class_contains_eq(cl, x)
        congruence_class_contains(cl, x)
    }
}

/// The integer greatest common divisor of two embedded naturals is the embedded natural one.
theorem int_gcd_from_nat(a: Nat, b: Nat) {
    Int.from_nat(a).gcd(Int.from_nat(b)) = Int.from_nat(a.gcd(b))
} by {
    (Int.from_nat(a).gcd(Int.from_nat(b))
        = Int.from_nat(abs(Int.from_nat(a)).gcd(abs(Int.from_nat(b)))))
    abs_from_nat(a)
    abs(Int.from_nat(a)) = a
    abs_from_nat(b)
    abs(Int.from_nat(b)) = b
    Int.from_nat(a).gcd(Int.from_nat(b)) = Int.from_nat(a.gcd(b))
}

/// Two congruence classes whose residue gap is a multiple of the gcd of the moduli meet.
///
/// The two-modulus Chinese remainder theorem in the form the disjointness criterion needs, and
/// without a coprimality hypothesis. Bezout writes the greatest common divisor as an integer
/// combination `u * m1 + v * m2` of the moduli; scaling by the multiple `k` that carries the gcd
/// to the residue gap, the integer `r1 - u * m1 * k` differs from `r1` by a multiple of `m1` and
/// from `r2` by `v * m2 * k`, a multiple of `m2`.
theorem classes_meet_of_gcd_divides(cl1: Pair[Nat, Nat], cl2: Pair[Nat, Nat]) {
    Int.from_nat(cl1.first).gcd(Int.from_nat(cl2.first)).divides(
        Int.from_nat(cl1.second) - Int.from_nat(cl2.second))
        implies exists(x: Int) {
            congruence_class_contains(cl1, x) and congruence_class_contains(cl2, x)
        }
} by {
    if Int.from_nat(cl1.first).gcd(Int.from_nat(cl2.first)).divides(
        Int.from_nat(cl1.second) - Int.from_nat(cl2.second)) {
        int_gcd_from_nat(cl1.first, cl2.first)
        (Int.from_nat(cl1.first).gcd(Int.from_nat(cl2.first))
            = Int.from_nat(cl1.first.gcd(cl2.first)))
        (Int.from_nat(cl1.first.gcd(cl2.first)).divides(
            Int.from_nat(cl1.second) - Int.from_nat(cl2.second)))
        exists(c: Int) {
            c * Int.from_nat(cl1.first.gcd(cl2.first))
                = Int.from_nat(cl1.second) - Int.from_nat(cl2.second)
        }
        let (k: Int) satisfy {
            k * Int.from_nat(cl1.first.gcd(cl2.first))
                = Int.from_nat(cl1.second) - Int.from_nat(cl2.second)
        }
        nat_bezout(cl1.first, cl2.first)
        exists(p: Int, q: Int) {
            p * Int.from_nat(cl1.first) + q * Int.from_nat(cl2.first)
                = Int.from_nat(cl1.first.gcd(cl2.first))
        }
        let (u: Int, v: Int) satisfy {
            u * Int.from_nat(cl1.first) + v * Int.from_nat(cl2.first)
                = Int.from_nat(cl1.first.gcd(cl2.first))
        }
        (k * (u * Int.from_nat(cl1.first) + v * Int.from_nat(cl2.first))
            = Int.from_nat(cl1.second) - Int.from_nat(cl2.second))
        (k * (u * Int.from_nat(cl1.first) + v * Int.from_nat(cl2.first))
            = (k * u) * Int.from_nat(cl1.first) + (k * v) * Int.from_nat(cl2.first))
        ((k * u) * Int.from_nat(cl1.first) + (k * v) * Int.from_nat(cl2.first)
            = Int.from_nat(cl1.second) - Int.from_nat(cl2.second))
        let w: Int = Int.from_nat(cl1.second) - (k * u) * Int.from_nat(cl1.first)
        (w = Int.from_nat(cl1.second) - (k * u) * Int.from_nat(cl1.first))
        (w = Int.from_nat(cl1.second) + -((k * u) * Int.from_nat(cl1.first)))
        (w - Int.from_nat(cl1.second) = w + -Int.from_nat(cl1.second))
        add_swap_inner[Int](Int.from_nat(cl1.second),
            -((k * u) * Int.from_nat(cl1.first)), -Int.from_nat(cl1.second), Int.0)
        ((Int.from_nat(cl1.second) + -((k * u) * Int.from_nat(cl1.first)))
            + (-Int.from_nat(cl1.second) + Int.0)
            = (Int.from_nat(cl1.second) + -Int.from_nat(cl1.second))
                + (-((k * u) * Int.from_nat(cl1.first)) + Int.0))
        (-Int.from_nat(cl1.second) + Int.0 = -Int.from_nat(cl1.second))
        (-((k * u) * Int.from_nat(cl1.first)) + Int.0
            = -((k * u) * Int.from_nat(cl1.first)))
        (Int.from_nat(cl1.second) + -Int.from_nat(cl1.second) = Int.0)
        (Int.0 + -((k * u) * Int.from_nat(cl1.first))
            = -((k * u) * Int.from_nat(cl1.first)))
        (w - Int.from_nat(cl1.second) = -((k * u) * Int.from_nat(cl1.first)))
        (-((k * u) * Int.from_nat(cl1.first)) = (-(k * u)) * Int.from_nat(cl1.first))
        ((-(k * u)) * Int.from_nat(cl1.first) = w - Int.from_nat(cl1.second))
        exists(c: Int) {
            c * Int.from_nat(cl1.first) = w - Int.from_nat(cl1.second)
        }
        Int.from_nat(cl1.first).divides(w - Int.from_nat(cl1.second))
        congruence_class_contains_of_divides(cl1, w)
        congruence_class_contains(cl1, w)
        (w - Int.from_nat(cl2.second) = w + -Int.from_nat(cl2.second))
        add_swap_inner[Int](Int.from_nat(cl1.second),
            -((k * u) * Int.from_nat(cl1.first)), -Int.from_nat(cl2.second), Int.0)
        ((Int.from_nat(cl1.second) + -((k * u) * Int.from_nat(cl1.first)))
            + (-Int.from_nat(cl2.second) + Int.0)
            = (Int.from_nat(cl1.second) + -Int.from_nat(cl2.second))
                + (-((k * u) * Int.from_nat(cl1.first)) + Int.0))
        (-Int.from_nat(cl2.second) + Int.0 = -Int.from_nat(cl2.second))
        ((Int.from_nat(cl1.second) + -Int.from_nat(cl2.second))
            = Int.from_nat(cl1.second) - Int.from_nat(cl2.second))
        (w - Int.from_nat(cl2.second)
            = (Int.from_nat(cl1.second) - Int.from_nat(cl2.second))
                + -((k * u) * Int.from_nat(cl1.first)))
        (w - Int.from_nat(cl2.second)
            = (Int.from_nat(cl1.second) - Int.from_nat(cl2.second))
                - (k * u) * Int.from_nat(cl1.first))
        (w - Int.from_nat(cl2.second)
            = ((k * u) * Int.from_nat(cl1.first) + (k * v) * Int.from_nat(cl2.first))
                - (k * u) * Int.from_nat(cl1.first))
        (((k * u) * Int.from_nat(cl1.first) + (k * v) * Int.from_nat(cl2.first))
            - (k * u) * Int.from_nat(cl1.first) = (k * v) * Int.from_nat(cl2.first))
        ((k * v) * Int.from_nat(cl2.first) = w - Int.from_nat(cl2.second))
        exists(c: Int) {
            c * Int.from_nat(cl2.first) = w - Int.from_nat(cl2.second)
        }
        Int.from_nat(cl2.first).divides(w - Int.from_nat(cl2.second))
        congruence_class_contains_of_divides(cl2, w)
        congruence_class_contains(cl2, w)
        exists(x: Int) {
            congruence_class_contains(cl1, x) and congruence_class_contains(cl2, x)
        }
    }
}

/// Disjoint classes have a residue gap missed by the gcd of their moduli.
///
/// The converse of `classes_disjoint_of_gcd_not_divides`, completing the criterion: two
/// congruence classes are disjoint exactly when the greatest common divisor of their moduli
/// fails to divide the gap between their residues.
theorem gcd_not_divides_of_classes_disjoint(cl1: Pair[Nat, Nat], cl2: Pair[Nat, Nat]) {
    classes_disjoint(cl1, cl2)
        implies not Int.from_nat(cl1.first).gcd(Int.from_nat(cl2.first)).divides(
            Int.from_nat(cl1.second) - Int.from_nat(cl2.second))
} by {
    if classes_disjoint(cl1, cl2) {
        if Int.from_nat(cl1.first).gcd(Int.from_nat(cl2.first)).divides(
            Int.from_nat(cl1.second) - Int.from_nat(cl2.second)) {
            classes_meet_of_gcd_divides(cl1, cl2)
            exists(x: Int) {
                congruence_class_contains(cl1, x) and congruence_class_contains(cl2, x)
            }
            let (w: Int) satisfy {
                congruence_class_contains(cl1, w) and congruence_class_contains(cl2, w)
            }
            classes_disjoint_apply(cl1, cl2, w)
            not (congruence_class_contains(cl1, w) and congruence_class_contains(cl2, w))
            false
        }
        not Int.from_nat(cl1.first).gcd(Int.from_nat(cl2.first)).divides(
            Int.from_nat(cl1.second) - Int.from_nat(cl2.second))
    }
}

/// Two congruence classes are disjoint exactly when the gcd of their moduli misses the residue
/// gap.
///
/// The criterion in closed form, both directions together.
theorem classes_disjoint_iff_gcd_not_divides(cl1: Pair[Nat, Nat], cl2: Pair[Nat, Nat]) {
    classes_disjoint(cl1, cl2)
        = not Int.from_nat(cl1.first).gcd(Int.from_nat(cl2.first)).divides(
            Int.from_nat(cl1.second) - Int.from_nat(cl2.second))
} by {
    if classes_disjoint(cl1, cl2) {
        gcd_not_divides_of_classes_disjoint(cl1, cl2)
        not Int.from_nat(cl1.first).gcd(Int.from_nat(cl2.first)).divides(
            Int.from_nat(cl1.second) - Int.from_nat(cl2.second))
    }
    if not Int.from_nat(cl1.first).gcd(Int.from_nat(cl2.first)).divides(
        Int.from_nat(cl1.second) - Int.from_nat(cl2.second)) {
        classes_disjoint_of_gcd_not_divides(cl1, cl2)
        classes_disjoint(cl1, cl2)
    }
    (classes_disjoint(cl1, cl2)
        implies not Int.from_nat(cl1.first).gcd(Int.from_nat(cl2.first)).divides(
            Int.from_nat(cl1.second) - Int.from_nat(cl2.second)))
    ((not Int.from_nat(cl1.first).gcd(Int.from_nat(cl2.first)).divides(
        Int.from_nat(cl1.second) - Int.from_nat(cl2.second)))
        implies classes_disjoint(cl1, cl2))
    (classes_disjoint(cl1, cl2)
        = not Int.from_nat(cl1.first).gcd(Int.from_nat(cl2.first)).divides(
            Int.from_nat(cl1.second) - Int.from_nat(cl2.second)))
}
