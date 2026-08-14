from algebra.add_comm_group import AddCommGroup
from comm_ring import CommRing
from ordered_field import OrderedField
from algebra.add_ordered_group import add_inequality
from algebra.ring.ring import mul_zero_left, mul_zero_right
from geometry.point3 import Point3, point3_zero, point3_ext,
    point3_sub_self, point3_add_comm, point3_dot_comm,
    point3_cross_self, point3_cross_zero_left, point3_cross_zero_right,
    point3_cross_x, point3_cross_y, point3_cross_z,
    point3_smul_x, point3_smul_y, point3_smul_z

/// The two middle terms in a four-term sum may be exchanged.
theorem point3_add_pair_rearrange[T: AddCommGroup](a: T, b: T, c: T, d: T) {
    (a + b) + (c + d) = (a + c) + (b + d)
} by {
    (c + b) + d = c + (b + d)
}

/// The last term of a four-term sum may be moved next to the first.
theorem point3_add_four_last_next_to_first[T: AddCommGroup](a: T, b: T, c: T, d: T) {
    a + b + c + d = (a + d) + (b + c)
} by {
    a + b + c + d = ((a + b) + c) + d
}

/// Subtraction through an intermediate element decomposes a difference.
theorem point3_scalar_sub_through[T: AddCommGroup](x: T, y: T, z: T) {
    (y - x) + (z - y) = z - x
} by {
    point3_add_four_last_next_to_first(y, -x, z, -y)
}

/// Reversing a scalar difference negates it.
theorem point3_scalar_sub_reverse_neg[T: AddCommGroup](x: T, y: T) {
    x - y = -(y - x)
} by {
}

/// A difference of sums may be regrouped as a sum of differences.
theorem point3_scalar_sub_pair_rearrange[T: AddCommGroup](a: T, b: T, c: T, d: T) {
    (a + b) - (c + d) = (a - c) + (b - d)
} by {
    point3_add_pair_rearrange(a, b, -c, -d)
}

/// A sum of differences may be regrouped as a difference of sums.
theorem point3_scalar_rev_sub_pair[T: AddCommGroup](a: T, b: T, c: T, d: T) {
    (a - c) + (b - d) = (a + b) - (c + d)
} by {
}

/// A difference of differences is the sum of the outer terms.
theorem point3_scalar_sub_of_sub[T: AddCommGroup](a: T, b: T, c: T, d: T) {
    (a - b) - (c - d) = (a - b) + (d - c)
} by {
    (a - b) - (c - d) = (a - b) + -(c - d)
    -(c - d) = d - c
    (a - b) + -(c - d) = (a - b) + (d - c)
}

/// Removing a common scalar base from two differences gives the endpoint difference.
theorem point3_scalar_sub_same_base[T: AddCommGroup](a: T, b: T, c: T) {
    (c - a) - (b - a) = c - b
} by {
    (c - a) - (b - a) = (c + -a) + (a + -b)
    (c + -a) + (a + -b) = c + (-a + a) + -b
    -a + a = T.0
}

/// A common additive term may be cancelled under subtraction.
theorem point3_scalar_add_sub_cancel[T: AddCommGroup](x: T, y: T, z: T) {
    (x + y) - (x + z) = y - z
} by {
    point3_scalar_sub_pair_rearrange(x, y, x, z)
    (x + y) - (x + z) = (x - x) + (y - z)
    x - x = T.0
    (x - x) + (y - z) = y - z
}

/// A common additive term may be cancelled under subtraction with three terms.
theorem point3_scalar_add_sub_cancel3[T: AddCommGroup](x: T, y: T, z: T, a: T, b: T) {
    (x + y + a) - (x + z + b) = (y + a) - (z + b)
} by {
    point3_scalar_sub_pair_rearrange(x, y + a, x, z + b)
    (x + (y + a)) - (x + (z + b)) = (x - x) + ((y + a) - (z + b))
    x - x = T.0
    (x - x) + ((y + a) - (z + b)) = (y + a) - (z + b)
    (x + y + a) - (x + z + b) = (x + (y + a)) - (x + (z + b))
}

/// Three cyclic differences sum to zero.
theorem point3_scalar_cyclic_cancel[T: AddCommGroup](a: T, b: T, c: T) {
    (a - b) + (c - a) + (b - c) = T.0
} by {
    (a - b) + (c - a) + (b - c) = ((a - b) + (c - a)) + (b - c)
    (a - b) + (c - a) = (a + c) - (b + a)
    (a + c) - (b + a) = c - b
    (c - b) + (b - c) = T.0
}

/// The square of a sum, with the two cross terms written separately.
theorem point3_ring_square_add[T: CommRing](x: T, y: T) {
    (x + y) * (x + y) = x * x + x * y + (x * y + y * y)
} by {
    (x * x + y * x) + (x * y + y * y) = x * x + x * y + (x * y + y * y)
}

/// The square of a difference, with the two cross terms written separately.
theorem point3_ring_square_sub[T: CommRing](x: T, y: T) {
    (x - y) * (x - y) = x * x - x * y - y * x + y * y
} by {
    point3_ring_square_add(x, -y)
}

/// A scalar distributes over a difference on the left.
theorem point3_scalar_dist_sub_left[T: CommRing](x: T, a: T, b: T) {
    x * (a - b) = x * a - x * b
} by {
}

/// A scalar distributes over a difference on the right.
theorem point3_scalar_dist_sub_right[T: CommRing](a: T, b: T, x: T) {
    (a - b) * x = a * x - b * x
} by {
}

/// A scalar distributes over a three-term sum.
theorem point3_scalar_dist_sum3[T: CommRing](x: T, a: T, b: T, c: T) {
    x * (a + b + c) = x * a + x * b + x * c
} by {
    x * (a + b + c) = x * ((a + b) + c)
    x * ((a + b) + c) = x * (a + b) + x * c
    x * (a + b) = x * a + x * b
    x * (a + b) + x * c = x * a + x * b + x * c
}

/// A product of three factors may be regrouped by the second factor.
theorem point3_scalar_prod_rearrange3[T: CommRing](x: T, y: T, z: T) {
    x * (y * z) = y * x * z
} by {
}

attributes Point3[T: CommRing] {
    /// True when two points are orthogonal as coordinate vectors.
    define orthogonal(self, other: Point3[T]) -> Bool {
        self.dot(other) = T.0
    }
}

/// Reversing a point difference negates it.
theorem point3_sub_reverse_neg[T: AddCommGroup](p: Point3[T], q: Point3[T]) {
    p.sub(q) = q.sub(p).neg
} by {
    let lhs = p.sub(q)
    let rhs = q.sub(p).neg
    point3_scalar_sub_reverse_neg(p.x, q.x)
    lhs.x = rhs.x
    point3_scalar_sub_reverse_neg(p.y, q.y)
    lhs.y = rhs.y
    point3_scalar_sub_reverse_neg(p.z, q.z)
    lhs.z = rhs.z
    point3_ext(lhs, rhs)
}

/// A point difference is addition of the additive inverse.
theorem point3_sub_eq_add_neg[T: AddCommGroup](p: Point3[T], q: Point3[T]) {
    p.sub(q) = p.add(q.neg)
} by {
    let lhs = p.sub(q)
    let rhs = p.add(q.neg)
    lhs.x = p.x - q.x
    rhs.x = p.x + -q.x
    lhs.x = rhs.x
    lhs.y = p.y - q.y
    rhs.y = p.y + -q.y
    lhs.y = rhs.y
    lhs.z = p.z - q.z
    rhs.z = p.z + -q.z
    lhs.z = rhs.z
    point3_ext(lhs, rhs)
}

/// Subtracting sums decomposes into the sum of subtractions.
theorem point3_sub_add_sub[T: AddCommGroup](a: Point3[T], b: Point3[T], c: Point3[T], d: Point3[T]) {
    a.add(b).sub(c.add(d)) = a.sub(c).add(b.sub(d))
} by {
    let lhs = a.add(b).sub(c.add(d))
    let rhs = a.sub(c).add(b.sub(d))
    lhs.x = (a.x + b.x) - (c.x + d.x)
    rhs.x = (a.x - c.x) + (b.x - d.x)
    point3_scalar_sub_pair_rearrange(a.x, b.x, c.x, d.x)
    lhs.x = rhs.x
    lhs.y = (a.y + b.y) - (c.y + d.y)
    rhs.y = (a.y - c.y) + (b.y - d.y)
    point3_scalar_sub_pair_rearrange(a.y, b.y, c.y, d.y)
    lhs.y = rhs.y
    lhs.z = (a.z + b.z) - (c.z + d.z)
    rhs.z = (a.z - c.z) + (b.z - d.z)
    point3_scalar_sub_pair_rearrange(a.z, b.z, c.z, d.z)
    lhs.z = rhs.z
    point3_ext(lhs, rhs)
}

/// Subtraction through an intermediate point decomposes a displacement.
theorem point3_sub_through[T: AddCommGroup](a: Point3[T], b: Point3[T], c: Point3[T]) {
    b.sub(a).add(c.sub(b)) = c.sub(a)
} by {
    let lhs = b.sub(a).add(c.sub(b))
    let rhs = c.sub(a)
    lhs.x = (b.x - a.x) + (c.x - b.x)
    rhs.x = c.x - a.x
    point3_scalar_sub_through(a.x, b.x, c.x)
    lhs.x = rhs.x
    lhs.y = (b.y - a.y) + (c.y - b.y)
    rhs.y = c.y - a.y
    point3_scalar_sub_through(a.y, b.y, c.y)
    lhs.y = rhs.y
    lhs.z = (b.z - a.z) + (c.z - b.z)
    rhs.z = c.z - a.z
    point3_scalar_sub_through(a.z, b.z, c.z)
    lhs.z = rhs.z
    point3_ext(lhs, rhs)
}

/// Subtracting the origin leaves a point unchanged.
theorem point3_sub_zero_right[T: AddCommGroup](p: Point3[T]) {
    p.sub(point3_zero[T]) = p
} by {
    point3_add_comm(p, point3_zero[T])
    let lhs = p.sub(point3_zero[T])
    lhs.x = p.x - T.0
    p.x - T.0 = p.x
    lhs.x = p.x
    lhs.y = p.y - T.0
    p.y - T.0 = p.y
    lhs.y = p.y
    lhs.z = p.z - T.0
    p.z - T.0 = p.z
    lhs.z = p.z
    point3_ext(lhs, p)
}

/// Removing a common base from two displacements gives the displacement between endpoints.
theorem point3_sub_same_base[T: AddCommGroup](a: Point3[T], b: Point3[T], c: Point3[T]) {
    c.sub(a).sub(b.sub(a)) = c.sub(b)
} by {
    let lhs = c.sub(a).sub(b.sub(a))
    let rhs = c.sub(b)
    lhs.x = (c.x - a.x) - (b.x - a.x)
    rhs.x = c.x - b.x
    point3_scalar_sub_same_base(a.x, b.x, c.x)
    (c.x - a.x) - (b.x - a.x) = c.x - b.x
    lhs.x = rhs.x
    lhs.y = (c.y - a.y) - (b.y - a.y)
    rhs.y = c.y - b.y
    point3_scalar_sub_same_base(a.y, b.y, c.y)
    (c.y - a.y) - (b.y - a.y) = c.y - b.y
    lhs.y = rhs.y
    lhs.z = (c.z - a.z) - (b.z - a.z)
    rhs.z = c.z - b.z
    point3_scalar_sub_same_base(a.z, b.z, c.z)
    (c.z - a.z) - (b.z - a.z) = c.z - b.z
    lhs.z = rhs.z
    point3_ext(lhs, rhs)
}

/// The dot product with the origin on the left is zero.
theorem point3_dot_zero_left[T: CommRing](p: Point3[T]) {
    point3_zero[T].dot(p) = T.0
} by {
    mul_zero_left[T](p.x)
    mul_zero_left[T](p.y)
    mul_zero_left[T](p.z)
}

/// The dot product with the origin on the right is zero.
theorem point3_dot_zero_right[T: CommRing](p: Point3[T]) {
    p.dot(point3_zero[T]) = T.0
} by {
    point3_dot_zero_left(p)
}

/// Negating the left argument negates a dot product.
theorem point3_dot_neg_left[T: CommRing](p: Point3[T], q: Point3[T]) {
    p.neg.dot(q) = -p.dot(q)
} by {
    -(p.x * q.x + p.y * q.y + p.z * q.z) =
        -(p.x * q.x) + -(p.y * q.y) + -(p.z * q.z)
}

/// Negating the right argument negates a dot product.
theorem point3_dot_neg_right[T: CommRing](p: Point3[T], q: Point3[T]) {
    p.dot(q.neg) = -p.dot(q)
} by {
    point3_dot_neg_left(q, p)
}

/// The dot product distributes over addition on the left.
theorem point3_dot_add_left[T: CommRing](a: Point3[T], b: Point3[T], c: Point3[T]) {
    a.add(b).dot(c) = a.dot(c) + b.dot(c)
} by {
    let x = a.x * c.x
    let y = b.x * c.x
    let z = a.y * c.y
    let w = b.y * c.y
    let u = a.z * c.z
    let v = b.z * c.z
    (a.x + b.x) * c.x = x + y
    (a.y + b.y) * c.y = z + w
    (a.z + b.z) * c.z = u + v
    (x + y) + (z + w) + (u + v) = ((x + y) + (z + w)) + (u + v)
    point3_add_pair_rearrange(x, y, z, w)
    (x + y) + (z + w) = (x + z) + (y + w)
    point3_add_pair_rearrange(x + z, y + w, u, v)
    ((x + z) + (y + w)) + (u + v) = ((x + z) + u) + ((y + w) + v)
    ((x + z) + u) + ((y + w) + v) = (x + z + u) + (y + w + v)
}

/// The dot product distributes over addition on the right.
theorem point3_dot_add_right[T: CommRing](a: Point3[T], b: Point3[T], c: Point3[T]) {
    a.dot(b.add(c)) = a.dot(b) + a.dot(c)
} by {
    point3_dot_add_left(b, c, a)
}

/// The dot product distributes over subtraction on the left.
theorem point3_dot_sub_left[T: CommRing](a: Point3[T], b: Point3[T], c: Point3[T]) {
    a.sub(b).dot(c) = a.dot(c) - b.dot(c)
} by {
    point3_sub_eq_add_neg(a, b)
    point3_dot_add_left(a, b.neg, c)
    point3_dot_neg_left(b, c)
}

/// The dot product distributes over subtraction on the right.
theorem point3_dot_sub_right[T: CommRing](a: Point3[T], b: Point3[T], c: Point3[T]) {
    a.dot(b.sub(c)) = a.dot(b) - a.dot(c)
} by {
    point3_dot_sub_left(b, c, a)
}

/// A scalar may be factored out of the left argument of a dot product.
theorem point3_dot_smul_left[T: CommRing](scalar: T, a: Point3[T], b: Point3[T]) {
    a.smul(scalar).dot(b) = scalar * a.dot(b)
} by {
    scalar * (a.x * b.x) + scalar * (a.y * b.y) + scalar * (a.z * b.z) =
        scalar * (a.x * b.x + a.y * b.y + a.z * b.z)
}

/// A scalar may be factored out of the right argument of a dot product.
theorem point3_dot_smul_right[T: CommRing](scalar: T, a: Point3[T], b: Point3[T]) {
    a.dot(b.smul(scalar)) = scalar * a.dot(b)
} by {
    point3_dot_smul_left(scalar, b, a)
}

/// The squared norm of a scalar multiple scales by the square of the scalar.
theorem point3_norm_sq_smul[T: CommRing](scalar: T, p: Point3[T]) {
    p.smul(scalar).norm_sq = scalar * scalar * p.norm_sq
} by {
    point3_dot_smul_left(scalar, p, p.smul(scalar))
    point3_dot_smul_right(scalar, p, p)
}

/// The squared norm is nonnegative over an ordered field.
theorem point3_norm_sq_nonneg[T: OrderedField](p: Point3[T]) {
    p.norm_sq >= T.0
} by {
    T.0 <= p.x * p.x
    T.0 <= p.y * p.y
    T.0 <= p.z * p.z
    add_inequality[T](T.0, p.x * p.x, T.0, p.y * p.y)
    T.0 + T.0 <= p.x * p.x + p.y * p.y
    add_inequality[T](T.0 + T.0, p.x * p.x + p.y * p.y, T.0, p.z * p.z)
    T.0 + T.0 + T.0 <= (p.x * p.x + p.y * p.y) + p.z * p.z
    T.0 + T.0 + T.0 = T.0
    T.0 <= (p.x * p.x + p.y * p.y) + p.z * p.z
    p.norm_sq = p.x * p.x + p.y * p.y + p.z * p.z
}

/// The origin has zero squared norm.
theorem point3_norm_sq_zero[T: CommRing](p: Point3[T]) {
    p = point3_zero[T] implies p.norm_sq = T.0
} by {
    if p = point3_zero[T] {
        point3_dot_zero_left(p)
    }
}

/// Negation preserves squared norm.
theorem point3_norm_sq_neg[T: CommRing](p: Point3[T]) {
    p.neg.norm_sq = p.norm_sq
} by {
    point3_dot_neg_left(p, p.neg)
    point3_dot_neg_right(p, p)
}

/// The squared norm of a sum expands by the dot product.
theorem point3_norm_sq_add_expansion[T: CommRing](a: Point3[T], b: Point3[T]) {
    a.add(b).norm_sq = a.norm_sq + b.norm_sq + a.dot(b) + a.dot(b)
} by {
    point3_dot_add_left(a, b, a.add(b))
    a.add(b).dot(a.add(b)) = a.dot(a.add(b)) + b.dot(a.add(b))
    point3_dot_add_right(a, a, b)
    a.dot(a.add(b)) = a.dot(a) + a.dot(b)
    point3_dot_add_right(b, a, b)
    b.dot(a.add(b)) = b.dot(a) + b.dot(b)
    point3_dot_comm(b, a)
    b.dot(a) = a.dot(b)
    a.add(b).dot(a.add(b)) =
        (a.dot(a) + a.dot(b)) + (a.dot(b) + b.dot(b))
    (a.dot(a) + a.dot(b)) + (a.dot(b) + b.dot(b)) =
        a.norm_sq + b.norm_sq + a.dot(b) + a.dot(b)
}

/// The squared norm of a difference expands by the dot product.
theorem point3_norm_sq_sub_expansion[T: CommRing](a: Point3[T], b: Point3[T]) {
    a.sub(b).norm_sq = a.norm_sq + b.norm_sq - a.dot(b) - a.dot(b)
} by {
    point3_sub_eq_add_neg(a, b)
    point3_norm_sq_add_expansion(a, b.neg)
    point3_norm_sq_neg(b)
    point3_dot_neg_right(a, b)
}

/// The squared distance from a point to itself is zero.
theorem point3_dist_sq_self[T: CommRing](p: Point3[T]) {
    p.dist_sq(p) = T.0
} by {
    point3_sub_self(p)
}

/// Squared distance is symmetric.
theorem point3_dist_sq_comm[T: CommRing](p: Point3[T], q: Point3[T]) {
    p.dist_sq(q) = q.dist_sq(p)
} by {
    point3_sub_reverse_neg(p, q)
    point3_norm_sq_neg(q.sub(p))
}

/// Squared distance is the squared norm of a difference.
theorem point3_dist_sq_eq_norm_sq_sub[T: CommRing](p: Point3[T], q: Point3[T]) {
    p.dist_sq(q) = p.sub(q).norm_sq
}

/// Squared distance to the origin is the squared norm.
theorem point3_dist_sq_zero_right[T: CommRing](p: Point3[T]) {
    p.dist_sq(point3_zero[T]) = p.norm_sq
} by {
    point3_sub_zero_right(p)
}

/// Negating the left argument negates a cross product.
theorem point3_cross_neg_left[T: CommRing](a: Point3[T], b: Point3[T]) {
    a.neg.cross(b) = a.cross(b).neg
} by {
    let lhs = a.neg.cross(b)
    let rhs = a.cross(b).neg
    let p = a.y * b.z
    let q = a.z * b.y
    let r = a.z * b.x
    let s = a.x * b.z
    let t = a.x * b.y
    let u = a.y * b.x
    point3_cross_x(a.neg, b)
    lhs.x = (-a.y) * b.z - (-a.z) * b.y
    (-a.y) * b.z = -p
    (-a.z) * b.y = -q
    -p - -q = -(p - q)
    lhs.x = -(a.y * b.z - a.z * b.y)
    point3_cross_x(a, b)
    rhs.x = -(a.y * b.z - a.z * b.y)
    lhs.x = rhs.x
    point3_cross_y(a.neg, b)
    lhs.y = (-a.z) * b.x - (-a.x) * b.z
    (-a.z) * b.x = -r
    (-a.x) * b.z = -s
    -r - -s = -(r - s)
    lhs.y = -(a.z * b.x - a.x * b.z)
    point3_cross_y(a, b)
    rhs.y = -(a.z * b.x - a.x * b.z)
    lhs.y = rhs.y
    point3_cross_z(a.neg, b)
    lhs.z = (-a.x) * b.y - (-a.y) * b.x
    (-a.x) * b.y = -t
    (-a.y) * b.x = -u
    -t - -u = -(t - u)
    lhs.z = -(a.x * b.y - a.y * b.x)
    point3_cross_z(a, b)
    rhs.z = -(a.x * b.y - a.y * b.x)
    lhs.z = rhs.z
    point3_ext(lhs, rhs)
}

/// Negating the right argument negates a cross product.
theorem point3_cross_neg_right[T: CommRing](a: Point3[T], b: Point3[T]) {
    a.cross(b.neg) = a.cross(b).neg
} by {
    let lhs = a.cross(b.neg)
    let rhs = a.cross(b).neg
    let p = a.y * b.z
    let q = a.z * b.y
    let r = a.z * b.x
    let s = a.x * b.z
    let t = a.x * b.y
    let u = a.y * b.x
    point3_cross_x(a, b.neg)
    lhs.x = a.y * (-b.z) - a.z * (-b.y)
    a.y * (-b.z) = -p
    a.z * (-b.y) = -q
    -p - -q = -(p - q)
    lhs.x = -(a.y * b.z - a.z * b.y)
    point3_cross_x(a, b)
    rhs.x = -(a.y * b.z - a.z * b.y)
    lhs.x = rhs.x
    point3_cross_y(a, b.neg)
    lhs.y = a.z * (-b.x) - a.x * (-b.z)
    a.z * (-b.x) = -r
    a.x * (-b.z) = -s
    -r - -s = -(r - s)
    lhs.y = -(a.z * b.x - a.x * b.z)
    point3_cross_y(a, b)
    rhs.y = -(a.z * b.x - a.x * b.z)
    lhs.y = rhs.y
    point3_cross_z(a, b.neg)
    lhs.z = a.x * (-b.y) - a.y * (-b.x)
    a.x * (-b.y) = -t
    a.y * (-b.x) = -u
    -t - -u = -(t - u)
    lhs.z = -(a.x * b.y - a.y * b.x)
    point3_cross_z(a, b)
    rhs.z = -(a.x * b.y - a.y * b.x)
    lhs.z = rhs.z
    point3_ext(lhs, rhs)
}

/// Swapping the arguments negates a cross product.
theorem point3_cross_swap[T: CommRing](a: Point3[T], b: Point3[T]) {
    a.cross(b) = b.cross(a).neg
} by {
    let lhs = a.cross(b)
    let rhs = b.cross(a).neg
    let p = a.y * b.z
    let q = a.z * b.y
    let r = a.z * b.x
    let s = a.x * b.z
    let t = a.x * b.y
    let u = a.y * b.x
    point3_cross_x(a, b)
    lhs.x = a.y * b.z - a.z * b.y
    point3_cross_x(b, a)
    rhs.x = -(b.y * a.z - b.z * a.y)
    b.y * a.z = q
    b.z * a.y = p
    a.y * b.z - a.z * b.y = -(b.y * a.z - b.z * a.y)
    lhs.x = rhs.x
    point3_cross_y(a, b)
    lhs.y = a.z * b.x - a.x * b.z
    point3_cross_y(b, a)
    rhs.y = -(b.z * a.x - b.x * a.z)
    b.z * a.x = s
    b.x * a.z = r
    a.z * b.x - a.x * b.z = -(b.z * a.x - b.x * a.z)
    lhs.y = rhs.y
    point3_cross_z(a, b)
    lhs.z = a.x * b.y - a.y * b.x
    point3_cross_z(b, a)
    rhs.z = -(b.x * a.y - b.y * a.x)
    b.x * a.y = u
    b.y * a.x = t
    a.x * b.y - a.y * b.x = -(b.x * a.y - b.y * a.x)
    lhs.z = rhs.z
    point3_ext(lhs, rhs)
}

/// The cross product distributes over addition on the left.
theorem point3_cross_add_left[T: CommRing](a: Point3[T], b: Point3[T], c: Point3[T]) {
    a.add(b).cross(c) = a.cross(c).add(b.cross(c))
} by {
    let lhs = a.add(b).cross(c)
    let rhs = a.cross(c).add(b.cross(c))
    point3_cross_x(a.add(b), c)
    lhs.x = (a.y + b.y) * c.z - (a.z + b.z) * c.y
    point3_scalar_dist_sum3(c.z, a.y, b.y, T.0)
    (a.y + b.y) * c.z = a.y * c.z + b.y * c.z
    point3_scalar_dist_sum3(c.y, a.z, b.z, T.0)
    (a.z + b.z) * c.y = a.z * c.y + b.z * c.y
    point3_scalar_rev_sub_pair(a.y * c.z, b.y * c.z, a.z * c.y, b.z * c.y)
    (a.y * c.z - a.z * c.y) + (b.y * c.z - b.z * c.y) =
        (a.y * c.z + b.y * c.z) - (a.z * c.y + b.z * c.y)
    point3_cross_x(a, c)
    point3_cross_x(b, c)
    lhs.x = (a.y * c.z - a.z * c.y) + (b.y * c.z - b.z * c.y)
    lhs.x = rhs.x
    point3_cross_y(a.add(b), c)
    lhs.y = (a.z + b.z) * c.x - (a.x + b.x) * c.z
    point3_scalar_dist_sum3(c.x, a.z, b.z, T.0)
    (a.z + b.z) * c.x = a.z * c.x + b.z * c.x
    point3_scalar_dist_sum3(c.z, a.x, b.x, T.0)
    (a.x + b.x) * c.z = a.x * c.z + b.x * c.z
    point3_scalar_rev_sub_pair(a.z * c.x, b.z * c.x, a.x * c.z, b.x * c.z)
    (a.z * c.x - a.x * c.z) + (b.z * c.x - b.x * c.z) =
        (a.z * c.x + b.z * c.x) - (a.x * c.z + b.x * c.z)
    point3_cross_y(a, c)
    point3_cross_y(b, c)
    lhs.y = (a.z * c.x - a.x * c.z) + (b.z * c.x - b.x * c.z)
    lhs.y = rhs.y
    point3_cross_z(a.add(b), c)
    lhs.z = (a.x + b.x) * c.y - (a.y + b.y) * c.x
    point3_scalar_dist_sum3(c.y, a.x, b.x, T.0)
    (a.x + b.x) * c.y = a.x * c.y + b.x * c.y
    point3_scalar_dist_sum3(c.x, a.y, b.y, T.0)
    (a.y + b.y) * c.x = a.y * c.x + b.y * c.x
    point3_scalar_rev_sub_pair(a.x * c.y, b.x * c.y, a.y * c.x, b.y * c.x)
    (a.x * c.y - a.y * c.x) + (b.x * c.y - b.y * c.x) =
        (a.x * c.y + b.x * c.y) - (a.y * c.x + b.y * c.x)
    point3_cross_z(a, c)
    point3_cross_z(b, c)
    lhs.z = (a.x * c.y - a.y * c.x) + (b.x * c.y - b.y * c.x)
    lhs.z = rhs.z
    point3_ext(lhs, rhs)
}

/// The cross product distributes over addition on the right.
theorem point3_cross_add_right[T: CommRing](a: Point3[T], b: Point3[T], c: Point3[T]) {
    a.cross(b.add(c)) = a.cross(b).add(a.cross(c))
} by {
    let lhs = a.cross(b.add(c))
    let rhs = a.cross(b).add(a.cross(c))
    point3_cross_x(a, b.add(c))
    lhs.x = a.y * (b.z + c.z) - a.z * (b.y + c.y)
    point3_scalar_dist_sum3(a.y, b.z, c.z, T.0)
    a.y * (b.z + c.z) = a.y * b.z + a.y * c.z
    point3_scalar_dist_sum3(a.z, b.y, c.y, T.0)
    a.z * (b.y + c.y) = a.z * b.y + a.z * c.y
    point3_scalar_rev_sub_pair(a.y * b.z, a.y * c.z, a.z * b.y, a.z * c.y)
    (a.y * b.z - a.z * b.y) + (a.y * c.z - a.z * c.y) =
        (a.y * b.z + a.y * c.z) - (a.z * b.y + a.z * c.y)
    point3_cross_x(a, b)
    point3_cross_x(a, c)
    lhs.x = (a.y * b.z - a.z * b.y) + (a.y * c.z - a.z * c.y)
    lhs.x = rhs.x
    point3_cross_y(a, b.add(c))
    lhs.y = a.z * (b.x + c.x) - a.x * (b.z + c.z)
    point3_scalar_dist_sum3(a.z, b.x, c.x, T.0)
    a.z * (b.x + c.x) = a.z * b.x + a.z * c.x
    point3_scalar_dist_sum3(a.x, b.z, c.z, T.0)
    a.x * (b.z + c.z) = a.x * b.z + a.x * c.z
    point3_scalar_rev_sub_pair(a.z * b.x, a.z * c.x, a.x * b.z, a.x * c.z)
    (a.z * b.x - a.x * b.z) + (a.z * c.x - a.x * c.z) =
        (a.z * b.x + a.z * c.x) - (a.x * b.z + a.x * c.z)
    point3_cross_y(a, b)
    point3_cross_y(a, c)
    lhs.y = (a.z * b.x - a.x * b.z) + (a.z * c.x - a.x * c.z)
    lhs.y = rhs.y
    point3_cross_z(a, b.add(c))
    lhs.z = a.x * (b.y + c.y) - a.y * (b.x + c.x)
    point3_scalar_dist_sum3(a.x, b.y, c.y, T.0)
    a.x * (b.y + c.y) = a.x * b.y + a.x * c.y
    point3_scalar_dist_sum3(a.y, b.x, c.x, T.0)
    a.y * (b.x + c.x) = a.y * b.x + a.y * c.x
    point3_scalar_rev_sub_pair(a.x * b.y, a.x * c.y, a.y * b.x, a.y * c.x)
    (a.x * b.y - a.y * b.x) + (a.x * c.y - a.y * c.x) =
        (a.x * b.y + a.x * c.y) - (a.y * b.x + a.y * c.x)
    point3_cross_z(a, b)
    point3_cross_z(a, c)
    lhs.z = (a.x * b.y - a.y * b.x) + (a.x * c.y - a.y * c.x)
    lhs.z = rhs.z
    point3_ext(lhs, rhs)
}

/// A scalar may be factored out of the left argument of a cross product.
theorem point3_cross_smul_left[T: CommRing](scalar: T, a: Point3[T], b: Point3[T]) {
    a.smul(scalar).cross(b) = a.cross(b).smul(scalar)
} by {
    let lhs = a.smul(scalar).cross(b)
    let rhs = a.cross(b).smul(scalar)
    let p = a.y * b.z
    let q = a.z * b.y
    let r = a.z * b.x
    let s = a.x * b.z
    let t = a.x * b.y
    let u = a.y * b.x
    point3_cross_x(a.smul(scalar), b)
    lhs.x = (scalar * a.y) * b.z - (scalar * a.z) * b.y
    (scalar * a.y) * b.z = scalar * p
    (scalar * a.z) * b.y = scalar * q
    scalar * p - scalar * q = scalar * (p - q)
    lhs.x = scalar * (a.y * b.z - a.z * b.y)
    point3_cross_x(a, b)
    rhs.x = scalar * (a.y * b.z - a.z * b.y)
    lhs.x = rhs.x
    point3_cross_y(a.smul(scalar), b)
    lhs.y = (scalar * a.z) * b.x - (scalar * a.x) * b.z
    (scalar * a.z) * b.x = scalar * r
    (scalar * a.x) * b.z = scalar * s
    scalar * r - scalar * s = scalar * (r - s)
    lhs.y = scalar * (a.z * b.x - a.x * b.z)
    point3_cross_y(a, b)
    rhs.y = scalar * (a.z * b.x - a.x * b.z)
    lhs.y = rhs.y
    point3_cross_z(a.smul(scalar), b)
    lhs.z = (scalar * a.x) * b.y - (scalar * a.y) * b.x
    (scalar * a.x) * b.y = scalar * t
    (scalar * a.y) * b.x = scalar * u
    scalar * t - scalar * u = scalar * (t - u)
    lhs.z = scalar * (a.x * b.y - a.y * b.x)
    point3_cross_z(a, b)
    rhs.z = scalar * (a.x * b.y - a.y * b.x)
    lhs.z = rhs.z
    point3_ext(lhs, rhs)
}

/// A scalar may be factored out of the right argument of a cross product.
theorem point3_cross_smul_right[T: CommRing](scalar: T, a: Point3[T], b: Point3[T]) {
    a.cross(b.smul(scalar)) = a.cross(b).smul(scalar)
} by {
    let lhs = a.cross(b.smul(scalar))
    let rhs = a.cross(b).smul(scalar)
    let p = a.y * b.z
    let q = a.z * b.y
    let r = a.z * b.x
    let s = a.x * b.z
    let t = a.x * b.y
    let u = a.y * b.x
    point3_cross_x(a, b.smul(scalar))
    lhs.x = a.y * (scalar * b.z) - a.z * (scalar * b.y)
    a.y * (scalar * b.z) = scalar * p
    a.z * (scalar * b.y) = scalar * q
    scalar * p - scalar * q = scalar * (p - q)
    lhs.x = scalar * (a.y * b.z - a.z * b.y)
    point3_cross_x(a, b)
    rhs.x = scalar * (a.y * b.z - a.z * b.y)
    lhs.x = rhs.x
    point3_cross_y(a, b.smul(scalar))
    lhs.y = a.z * (scalar * b.x) - a.x * (scalar * b.z)
    a.z * (scalar * b.x) = scalar * r
    a.x * (scalar * b.z) = scalar * s
    scalar * r - scalar * s = scalar * (r - s)
    lhs.y = scalar * (a.z * b.x - a.x * b.z)
    point3_cross_y(a, b)
    rhs.y = scalar * (a.z * b.x - a.x * b.z)
    lhs.y = rhs.y
    point3_cross_z(a, b.smul(scalar))
    lhs.z = a.x * (scalar * b.y) - a.y * (scalar * b.x)
    a.x * (scalar * b.y) = scalar * t
    a.y * (scalar * b.x) = scalar * u
    scalar * t - scalar * u = scalar * (t - u)
    lhs.z = scalar * (a.x * b.y - a.y * b.x)
    point3_cross_z(a, b)
    rhs.z = scalar * (a.x * b.y - a.y * b.x)
    lhs.z = rhs.z
    point3_ext(lhs, rhs)
}

/// The cross product distributes over subtraction on the left.
theorem point3_cross_sub_left[T: CommRing](a: Point3[T], b: Point3[T], c: Point3[T]) {
    a.sub(b).cross(c) = a.cross(c).sub(b.cross(c))
} by {
    point3_sub_eq_add_neg(a, b)
    point3_cross_add_left(a, b.neg, c)
    point3_cross_neg_left(b, c)
}

/// The cross product distributes over subtraction on the right.
theorem point3_cross_sub_right[T: CommRing](a: Point3[T], b: Point3[T], c: Point3[T]) {
    a.cross(b.sub(c)) = a.cross(b).sub(a.cross(c))
} by {
    point3_sub_eq_add_neg(b, c)
    point3_cross_add_right(a, b, c.neg)
    point3_cross_neg_right(a, c)
}

/// Orthogonality is symmetric.
theorem point3_orthogonal_comm[T: CommRing](p: Point3[T], q: Point3[T]) {
    p.orthogonal(q) = q.orthogonal(p)
} by {
    point3_dot_comm(p, q)
}

/// Negating the left argument preserves orthogonality.
theorem point3_orthogonal_neg_left[T: CommRing](p: Point3[T], q: Point3[T]) {
    p.neg.orthogonal(q) = p.orthogonal(q)
} by {
    if p.neg.orthogonal(q) {
        point3_dot_neg_left(p, q)
        -p.dot(q) = T.0
        p.dot(q) = T.0
        p.orthogonal(q)
    }
    if p.orthogonal(q) {
        point3_dot_neg_left(p, q)
        -p.dot(q) = -T.0
        -T.0 = T.0
        p.neg.orthogonal(q)
    }
}

/// Negating the right argument preserves orthogonality.
theorem point3_orthogonal_neg_right[T: CommRing](p: Point3[T], q: Point3[T]) {
    p.orthogonal(q.neg) = p.orthogonal(q)
} by {
    q.orthogonal(p) = p.orthogonal(q)
}

/// Scaling the left argument by a scalar preserves orthogonality from an orthogonal pair.
theorem point3_orthogonal_smul_left[T: CommRing](scalar: T, p: Point3[T], q: Point3[T]) {
    p.orthogonal(q) implies p.smul(scalar).orthogonal(q)
} by {
    if p.orthogonal(q) {
        p.dot(q) = T.0
        point3_dot_smul_left(scalar, p, q)
        mul_zero_right[T](scalar)
        scalar * p.dot(q) = T.0
    }
}

/// Scaling the right argument by a scalar preserves orthogonality from an orthogonal pair.
theorem point3_orthogonal_smul_right[T: CommRing](scalar: T, p: Point3[T], q: Point3[T]) {
    p.orthogonal(q) implies p.orthogonal(q.smul(scalar))
} by {
    if p.orthogonal(q) {
        p.dot(q) = T.0
        point3_dot_smul_right(scalar, p, q)
        mul_zero_right[T](scalar)
        scalar * p.dot(q) = T.0
    }
}

/// Pythagoras for orthogonal coordinate vectors.
theorem point3_pythagoras_vectors[T: CommRing](p: Point3[T], q: Point3[T]) {
    p.orthogonal(q) implies p.add(q).norm_sq = p.norm_sq + q.norm_sq
} by {
    if p.orthogonal(q) {
        p.dot(q) = T.0
        point3_norm_sq_add_expansion(p, q)
        p.add(q).norm_sq = p.norm_sq + q.norm_sq
    }
}

/// Pythagoras for three points using the two successive displacements.
theorem point3_pythagoras_points_from_ab[T: CommRing](a: Point3[T], b: Point3[T], c: Point3[T]) {
    b.sub(a).orthogonal(c.sub(b)) implies
    c.sub(a).norm_sq = b.sub(a).norm_sq + c.sub(b).norm_sq
} by {
    if b.sub(a).orthogonal(c.sub(b)) {
        point3_sub_through(a, b, c)
        c.sub(a).norm_sq = b.sub(a).add(c.sub(b)).norm_sq
        point3_pythagoras_vectors(b.sub(a), c.sub(b))
    }
}

/// Pythagoras for three points with the first displacement reversed.
theorem point3_pythagoras_points[T: CommRing](a: Point3[T], b: Point3[T], c: Point3[T]) {
    a.sub(b).orthogonal(c.sub(b)) implies
    c.sub(a).norm_sq = b.sub(a).norm_sq + c.sub(b).norm_sq
} by {
    if a.sub(b).orthogonal(c.sub(b)) {
        point3_sub_reverse_neg(a, b)
        point3_orthogonal_neg_left(b.sub(a), c.sub(b))
        b.sub(a).orthogonal(c.sub(b))
        point3_pythagoras_points_from_ab(a, b, c)
    }
}

/// The cross product is orthogonal to its first argument.
theorem point3_cross_orthogonal_left[T: CommRing](u: Point3[T], v: Point3[T]) {
    u.cross(v).dot(u) = T.0
} by {
    let p = u.x * u.y * v.z
    let q = u.x * u.z * v.y
    let r = u.y * u.z * v.x
    point3_cross_x(u, v)
    point3_cross_y(u, v)
    point3_cross_z(u, v)
    u.cross(v).dot(u) = u.cross(v).x * u.x + u.cross(v).y * u.y + u.cross(v).z * u.z
    u.cross(v).x * u.x = (u.y * v.z - u.z * v.y) * u.x
    point3_scalar_dist_sub_right(u.y * v.z, u.z * v.y, u.x)
    (u.y * v.z - u.z * v.y) * u.x = (u.y * v.z) * u.x - (u.z * v.y) * u.x
    (u.y * v.z) * u.x = p
    (u.z * v.y) * u.x = q
    u.cross(v).x * u.x = p - q
    u.cross(v).y * u.y = (u.z * v.x - u.x * v.z) * u.y
    point3_scalar_dist_sub_right(u.z * v.x, u.x * v.z, u.y)
    (u.z * v.x - u.x * v.z) * u.y = (u.z * v.x) * u.y - (u.x * v.z) * u.y
    (u.z * v.x) * u.y = r
    (u.x * v.z) * u.y = p
    u.cross(v).y * u.y = r - p
    u.cross(v).z * u.z = (u.x * v.y - u.y * v.x) * u.z
    point3_scalar_dist_sub_right(u.x * v.y, u.y * v.x, u.z)
    (u.x * v.y - u.y * v.x) * u.z = (u.x * v.y) * u.z - (u.y * v.x) * u.z
    (u.x * v.y) * u.z = q
    (u.y * v.x) * u.z = r
    u.cross(v).z * u.z = q - r
    u.cross(v).dot(u) = (p - q) + (r - p) + (q - r)
    point3_scalar_cyclic_cancel(p, q, r)
    (p - q) + (r - p) + (q - r) = T.0
}

/// The cross product is orthogonal to its second argument.
theorem point3_cross_orthogonal_right[T: CommRing](u: Point3[T], v: Point3[T]) {
    u.cross(v).dot(v) = T.0
} by {
    point3_cross_swap(u, v)
    u.cross(v) = v.cross(u).neg
    point3_dot_neg_left(v.cross(u), v)
    v.cross(u).neg.dot(v) = -v.cross(u).dot(v)
    point3_cross_orthogonal_left(v, u)
    v.cross(u).dot(v) = T.0
    -T.0 = T.0
    u.cross(v).dot(v) = T.0
}

/// The scalar triple product is unchanged by cyclically rotating its arguments.
theorem point3_scalar_triple_rotate[T: CommRing](u: Point3[T], v: Point3[T], w: Point3[T]) {
    u.dot(v.cross(w)) = v.dot(w.cross(u))
} by {
    let lhs = u.dot(v.cross(w))
    let rhs = v.dot(w.cross(u))
    let p1 = u.x * v.y * w.z
    let p2 = u.x * v.z * w.y
    let p3 = u.y * v.z * w.x
    let p4 = u.y * v.x * w.z
    let p5 = u.z * v.x * w.y
    let p6 = u.z * v.y * w.x
    point3_cross_x(v, w)
    point3_cross_y(v, w)
    point3_cross_z(v, w)
    lhs = u.x * (v.y * w.z - v.z * w.y) +
        u.y * (v.z * w.x - v.x * w.z) +
        u.z * (v.x * w.y - v.y * w.x)
    point3_scalar_dist_sub_left(u.x, v.y * w.z, v.z * w.y)
    u.x * (v.y * w.z - v.z * w.y) = u.x * (v.y * w.z) - u.x * (v.z * w.y)
    point3_scalar_dist_sub_left(u.y, v.z * w.x, v.x * w.z)
    u.y * (v.z * w.x - v.x * w.z) = u.y * (v.z * w.x) - u.y * (v.x * w.z)
    point3_scalar_dist_sub_left(u.z, v.x * w.y, v.y * w.x)
    u.z * (v.x * w.y - v.y * w.x) = u.z * (v.x * w.y) - u.z * (v.y * w.x)
    u.x * (v.y * w.z) = p1
    u.x * (v.z * w.y) = p2
    u.y * (v.z * w.x) = p3
    u.y * (v.x * w.z) = p4
    u.z * (v.x * w.y) = p5
    u.z * (v.y * w.x) = p6
    lhs = (p1 - p2) + (p3 - p4) + (p5 - p6)
    point3_scalar_rev_sub_pair(p1, p3, p2, p4)
    (p1 - p2) + (p3 - p4) = (p1 + p3) - (p2 + p4)
    point3_scalar_rev_sub_pair(p1 + p3, p5, p2 + p4, p6)
    ((p1 + p3) - (p2 + p4)) + (p5 - p6) = (p1 + p3 + p5) - (p2 + p4 + p6)
    lhs = (p1 + p3 + p5) - (p2 + p4 + p6)
    point3_cross_x(w, u)
    point3_cross_y(w, u)
    point3_cross_z(w, u)
    rhs = v.x * (w.y * u.z - w.z * u.y) +
        v.y * (w.z * u.x - w.x * u.z) +
        v.z * (w.x * u.y - w.y * u.x)
    point3_scalar_dist_sub_left(v.x, w.y * u.z, w.z * u.y)
    v.x * (w.y * u.z - w.z * u.y) = v.x * (w.y * u.z) - v.x * (w.z * u.y)
    point3_scalar_dist_sub_left(v.y, w.z * u.x, w.x * u.z)
    v.y * (w.z * u.x - w.x * u.z) = v.y * (w.z * u.x) - v.y * (w.x * u.z)
    point3_scalar_dist_sub_left(v.z, w.x * u.y, w.y * u.x)
    v.z * (w.x * u.y - w.y * u.x) = v.z * (w.x * u.y) - v.z * (w.y * u.x)
    v.x * (w.y * u.z) = p5
    v.x * (w.z * u.y) = p4
    v.y * (w.z * u.x) = p1
    v.y * (w.x * u.z) = p6
    v.z * (w.x * u.y) = p3
    v.z * (w.y * u.x) = p2
    rhs = (p5 - p4) + (p1 - p6) + (p3 - p2)
    point3_scalar_rev_sub_pair(p5, p1, p4, p6)
    (p5 - p4) + (p1 - p6) = (p5 + p1) - (p4 + p6)
    point3_scalar_rev_sub_pair(p5 + p1, p3, p4 + p6, p2)
    ((p5 + p1) - (p4 + p6)) + (p3 - p2) = (p5 + p1 + p3) - (p4 + p6 + p2)
    rhs = (p5 + p1 + p3) - (p4 + p6 + p2)
    p1 + p3 + p5 = p5 + p1 + p3
    p2 + p4 + p6 = p4 + p6 + p2
    (p1 + p3 + p5) - (p2 + p4 + p6) = (p5 + p1 + p3) - (p4 + p6 + p2)
    lhs = rhs
}

/// The scalar triple product is unchanged by a double rotation of its arguments.
theorem point3_scalar_triple_rotate2[T: CommRing](u: Point3[T], v: Point3[T], w: Point3[T]) {
    u.dot(v.cross(w)) = w.dot(u.cross(v))
} by {
    point3_scalar_triple_rotate(u, v, w)
    point3_scalar_triple_rotate(v, w, u)
}

/// The scalar triple product may move the cross product to either argument.
theorem point3_scalar_triple_mixed[T: CommRing](u: Point3[T], v: Point3[T], w: Point3[T]) {
    u.cross(v).dot(w) = u.dot(v.cross(w))
} by {
    point3_scalar_triple_rotate2(u, v, w)
    u.dot(v.cross(w)) = w.dot(u.cross(v))
}

/// The vector triple product (Lagrange's formula).
theorem point3_vector_triple_product[T: CommRing](u: Point3[T], v: Point3[T], w: Point3[T]) {
    u.cross(v.cross(w)) = v.smul(u.dot(w)).sub(w.smul(u.dot(v)))
} by {
    let lhs = u.cross(v.cross(w))
    let rhs = v.smul(u.dot(w)).sub(w.smul(u.dot(v)))
    point3_cross_x(u, v.cross(w))
    lhs.x = u.y * (v.cross(w).z) - u.z * (v.cross(w).y)
    point3_cross_z(v, w)
    point3_cross_y(v, w)
    lhs.x = u.y * (v.x * w.y - v.y * w.x) - u.z * (v.z * w.x - v.x * w.z)
    point3_scalar_dist_sub_left(u.y, v.x * w.y, v.y * w.x)
    u.y * (v.x * w.y - v.y * w.x) = u.y * (v.x * w.y) - u.y * (v.y * w.x)
    point3_scalar_dist_sub_left(u.z, v.z * w.x, v.x * w.z)
    u.z * (v.z * w.x - v.x * w.z) = u.z * (v.z * w.x) - u.z * (v.x * w.z)
    u.y * (v.x * w.y) = u.y * v.x * w.y
    u.y * (v.y * w.x) = u.y * v.y * w.x
    u.z * (v.z * w.x) = u.z * v.z * w.x
    u.z * (v.x * w.z) = u.z * v.x * w.z
    lhs.x = (u.y * v.x * w.y - u.y * v.y * w.x) - (u.z * v.z * w.x - u.z * v.x * w.z)
    point3_scalar_sub_of_sub(u.y * v.x * w.y, u.y * v.y * w.x, u.z * v.z * w.x, u.z * v.x * w.z)
    lhs.x = (u.y * v.x * w.y - u.y * v.y * w.x) + (u.z * v.x * w.z - u.z * v.z * w.x)
    rhs.x = v.x * u.dot(w) - w.x * u.dot(v)
    u.dot(w) = u.x * w.x + u.y * w.y + u.z * w.z
    u.dot(v) = u.x * v.x + u.y * v.y + u.z * v.z
    point3_scalar_dist_sum3(v.x, u.x * w.x, u.y * w.y, u.z * w.z)
    v.x * (u.x * w.x + u.y * w.y + u.z * w.z) =
        v.x * (u.x * w.x) + v.x * (u.y * w.y) + v.x * (u.z * w.z)
    point3_scalar_dist_sum3(w.x, u.x * v.x, u.y * v.y, u.z * v.z)
    w.x * (u.x * v.x + u.y * v.y + u.z * v.z) =
        w.x * (u.x * v.x) + w.x * (u.y * v.y) + w.x * (u.z * v.z)
    rhs.x = (v.x * (u.x * w.x) + v.x * (u.y * w.y) + v.x * (u.z * w.z)) -
        (w.x * (u.x * v.x) + w.x * (u.y * v.y) + w.x * (u.z * v.z))
    v.x * (u.x * w.x) = u.x * v.x * w.x
    v.x * (u.y * w.y) = u.y * v.x * w.y
    v.x * (u.z * w.z) = u.z * v.x * w.z
    w.x * (u.x * v.x) = u.x * v.x * w.x
    w.x * (u.y * v.y) = u.y * v.y * w.x
    w.x * (u.z * v.z) = u.z * v.z * w.x
    rhs.x = (u.x * v.x * w.x + u.y * v.x * w.y + u.z * v.x * w.z) -
        (u.x * v.x * w.x + u.y * v.y * w.x + u.z * v.z * w.x)
    point3_scalar_add_sub_cancel3(
        u.x * v.x * w.x,
        u.y * v.x * w.y,
        u.y * v.y * w.x,
        u.z * v.x * w.z,
        u.z * v.z * w.x)
    rhs.x = (u.y * v.x * w.y + u.z * v.x * w.z) - (u.y * v.y * w.x + u.z * v.z * w.x)
    point3_scalar_rev_sub_pair(u.y * v.x * w.y, u.z * v.x * w.z, u.y * v.y * w.x, u.z * v.z * w.x)
    rhs.x = (u.y * v.x * w.y - u.y * v.y * w.x) + (u.z * v.x * w.z - u.z * v.z * w.x)
    lhs.x = rhs.x
    point3_cross_y(u, v.cross(w))
    lhs.y = u.z * (v.cross(w).x) - u.x * (v.cross(w).z)
    point3_cross_x(v, w)
    point3_cross_z(v, w)
    lhs.y = u.z * (v.y * w.z - v.z * w.y) - u.x * (v.x * w.y - v.y * w.x)
    point3_scalar_dist_sub_left(u.z, v.y * w.z, v.z * w.y)
    u.z * (v.y * w.z - v.z * w.y) = u.z * (v.y * w.z) - u.z * (v.z * w.y)
    point3_scalar_dist_sub_left(u.x, v.x * w.y, v.y * w.x)
    u.x * (v.x * w.y - v.y * w.x) = u.x * (v.x * w.y) - u.x * (v.y * w.x)
    u.z * (v.y * w.z) = u.z * v.y * w.z
    u.z * (v.z * w.y) = u.z * v.z * w.y
    u.x * (v.x * w.y) = u.x * v.x * w.y
    u.x * (v.y * w.x) = u.x * v.y * w.x
    lhs.y = (u.z * v.y * w.z - u.z * v.z * w.y) - (u.x * v.x * w.y - u.x * v.y * w.x)
    point3_scalar_sub_of_sub(u.z * v.y * w.z, u.z * v.z * w.y, u.x * v.x * w.y, u.x * v.y * w.x)
    lhs.y = (u.z * v.y * w.z - u.z * v.z * w.y) + (u.x * v.y * w.x - u.x * v.x * w.y)
    point3_smul_y(v, u.dot(w))
    v.smul(u.dot(w)).y = u.dot(w) * v.y
    point3_smul_y(w, u.dot(v))
    w.smul(u.dot(v)).y = u.dot(v) * w.y
    u.dot(w) * v.y = v.y * u.dot(w)
    u.dot(v) * w.y = w.y * u.dot(v)
    rhs.y = v.y * u.dot(w) - w.y * u.dot(v)
    point3_scalar_dist_sum3(v.y, u.x * w.x, u.y * w.y, u.z * w.z)
    v.y * (u.x * w.x + u.y * w.y + u.z * w.z) =
        v.y * (u.x * w.x) + v.y * (u.y * w.y) + v.y * (u.z * w.z)
    point3_scalar_dist_sum3(w.y, u.x * v.x, u.y * v.y, u.z * v.z)
    w.y * (u.x * v.x + u.y * v.y + u.z * v.z) =
        w.y * (u.x * v.x) + w.y * (u.y * v.y) + w.y * (u.z * v.z)
    rhs.y = (v.y * (u.x * w.x) + v.y * (u.y * w.y) + v.y * (u.z * w.z)) -
        (w.y * (u.x * v.x) + w.y * (u.y * v.y) + w.y * (u.z * v.z))
    v.y * (u.x * w.x) = u.x * v.y * w.x
    v.y * (u.y * w.y) = u.y * v.y * w.y
    v.y * (u.z * w.z) = u.z * v.y * w.z
    w.y * (u.x * v.x) = u.x * v.x * w.y
    w.y * (u.y * v.y) = u.y * v.y * w.y
    w.y * (u.z * v.z) = u.z * v.z * w.y
    rhs.y = (u.x * v.y * w.x + u.y * v.y * w.y + u.z * v.y * w.z) -
        (u.x * v.x * w.y + u.y * v.y * w.y + u.z * v.z * w.y)
    point3_scalar_add_sub_cancel3(
        u.y * v.y * w.y,
        u.z * v.y * w.z,
        u.z * v.z * w.y,
        u.x * v.y * w.x,
        u.x * v.x * w.y)
    rhs.y = (u.z * v.y * w.z + u.x * v.y * w.x) - (u.z * v.z * w.y + u.x * v.x * w.y)
    point3_scalar_rev_sub_pair(u.z * v.y * w.z, u.x * v.y * w.x, u.z * v.z * w.y, u.x * v.x * w.y)
    rhs.y = (u.z * v.y * w.z - u.z * v.z * w.y) + (u.x * v.y * w.x - u.x * v.x * w.y)
    lhs.y = rhs.y
    point3_cross_z(u, v.cross(w))
    lhs.z = u.x * (v.cross(w).y) - u.y * (v.cross(w).x)
    point3_cross_y(v, w)
    point3_cross_x(v, w)
    lhs.z = u.x * (v.z * w.x - v.x * w.z) - u.y * (v.y * w.z - v.z * w.y)
    point3_scalar_dist_sub_left(u.x, v.z * w.x, v.x * w.z)
    u.x * (v.z * w.x - v.x * w.z) = u.x * (v.z * w.x) - u.x * (v.x * w.z)
    point3_scalar_dist_sub_left(u.y, v.y * w.z, v.z * w.y)
    u.y * (v.y * w.z - v.z * w.y) = u.y * (v.y * w.z) - u.y * (v.z * w.y)
    u.x * (v.z * w.x) = u.x * v.z * w.x
    u.x * (v.x * w.z) = u.x * v.x * w.z
    u.y * (v.y * w.z) = u.y * v.y * w.z
    u.y * (v.z * w.y) = u.y * v.z * w.y
    lhs.z = (u.x * v.z * w.x - u.x * v.x * w.z) - (u.y * v.y * w.z - u.y * v.z * w.y)
    point3_scalar_sub_of_sub(u.x * v.z * w.x, u.x * v.x * w.z, u.y * v.y * w.z, u.y * v.z * w.y)
    lhs.z = (u.x * v.z * w.x - u.x * v.x * w.z) + (u.y * v.z * w.y - u.y * v.y * w.z)
    point3_smul_z(v, u.dot(w))
    v.smul(u.dot(w)).z = u.dot(w) * v.z
    point3_smul_z(w, u.dot(v))
    w.smul(u.dot(v)).z = u.dot(v) * w.z
    u.dot(w) * v.z = v.z * u.dot(w)
    u.dot(v) * w.z = w.z * u.dot(v)
    rhs.z = v.z * u.dot(w) - w.z * u.dot(v)
    point3_scalar_dist_sum3(v.z, u.x * w.x, u.y * w.y, u.z * w.z)
    v.z * (u.x * w.x + u.y * w.y + u.z * w.z) =
        v.z * (u.x * w.x) + v.z * (u.y * w.y) + v.z * (u.z * w.z)
    point3_scalar_dist_sum3(w.z, u.x * v.x, u.y * v.y, u.z * v.z)
    w.z * (u.x * v.x + u.y * v.y + u.z * v.z) =
        w.z * (u.x * v.x) + w.z * (u.y * v.y) + w.z * (u.z * v.z)
    rhs.z = (v.z * (u.x * w.x) + v.z * (u.y * w.y) + v.z * (u.z * w.z)) -
        (w.z * (u.x * v.x) + w.z * (u.y * v.y) + w.z * (u.z * v.z))
    v.z * (u.x * w.x) = u.x * v.z * w.x
    v.z * (u.y * w.y) = u.y * v.z * w.y
    v.z * (u.z * w.z) = u.z * v.z * w.z
    w.z * (u.x * v.x) = u.x * v.x * w.z
    w.z * (u.y * v.y) = u.y * v.y * w.z
    w.z * (u.z * v.z) = u.z * v.z * w.z
    rhs.z = (u.x * v.z * w.x + u.y * v.z * w.y + u.z * v.z * w.z) -
        (u.x * v.x * w.z + u.y * v.y * w.z + u.z * v.z * w.z)
    point3_scalar_add_sub_cancel3(
        u.z * v.z * w.z,
        u.x * v.z * w.x,
        u.x * v.x * w.z,
        u.y * v.z * w.y,
        u.y * v.y * w.z)
    rhs.z = (u.x * v.z * w.x + u.y * v.z * w.y) - (u.x * v.x * w.z + u.y * v.y * w.z)
    point3_scalar_rev_sub_pair(u.x * v.z * w.x, u.y * v.z * w.y, u.x * v.x * w.z, u.y * v.y * w.z)
    rhs.z = (u.x * v.z * w.x - u.x * v.x * w.z) + (u.y * v.z * w.y - u.y * v.y * w.z)
    lhs.z = rhs.z
    point3_ext(lhs, rhs)
}

/// Lagrange's identity: the squared norm of a cross product is the squared
/// norms product minus the squared dot product.
theorem point3_cross_norm_sq_identity[T: CommRing](u: Point3[T], v: Point3[T]) {
    u.cross(v).norm_sq = u.norm_sq * v.norm_sq - u.dot(v) * u.dot(v)
} by {
    point3_scalar_triple_mixed(u, v, u.cross(v))
    u.cross(v).dot(u.cross(v)) = u.dot(v.cross(u.cross(v)))
    point3_vector_triple_product(v, u, v)
    v.cross(u.cross(v)) = u.smul(v.dot(v)).sub(v.smul(v.dot(u)))
    point3_dot_sub_right(u, u.smul(v.dot(v)), v.smul(v.dot(u)))
    u.dot(u.smul(v.dot(v)).sub(v.smul(v.dot(u)))) =
        u.dot(u.smul(v.dot(v))) - u.dot(v.smul(v.dot(u)))
    point3_dot_smul_right(v.dot(v), u, u)
    u.dot(u.smul(v.dot(v))) = v.dot(v) * u.dot(u)
    point3_dot_smul_right(v.dot(u), u, v)
    u.dot(v.smul(v.dot(u))) = v.dot(u) * u.dot(v)
    point3_dot_comm(v, u)
    v.dot(u) = u.dot(v)
    u.dot(v.cross(u.cross(v))) = v.dot(v) * u.dot(u) - u.dot(v) * u.dot(v)
    u.cross(v).norm_sq = u.cross(v).dot(u.cross(v))
    u.cross(v).dot(u.cross(v)) = v.dot(v) * u.dot(u) - u.dot(v) * u.dot(v)
    v.dot(v) * u.dot(u) = u.norm_sq * v.norm_sq
    u.cross(v).norm_sq = u.norm_sq * v.norm_sq - u.dot(v) * u.dot(v)
}
