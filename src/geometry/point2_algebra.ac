from algebra.add_comm_group import AddCommGroup
from comm_ring import CommRing
from ordered_field import OrderedField
from algebra.ring.ring import mul_zero_left, mul_zero_right
from geometry.point2 import Point2, point2_zero, point2_ext,
    point2_sub_self, point2_add_comm, point2_dot_comm,
    point2_cross_self, point2_cross_zero_left, point2_cross_zero_right

/// The square of a sum, with the two cross terms written separately.
theorem point2_ring_square_add[T: CommRing](x: T, y: T) {
    (x + y) * (x + y) = x * x + x * y + (x * y + y * y)
} by {
    (x * x + y * x) + (x * y + y * y) = x * x + x * y + (x * y + y * y)
}

/// Repeated middle terms may be grouped in a four-term sum.
theorem point2_add_repeated_middle[T: AddCommGroup](a: T, b: T, c: T) {
    a + b + (b + c) = a + (b + b) + c
} by {
    a + b + (b + c) = (a + b) + (b + c)
}

/// Repeated terms may be paired in either order.
theorem point2_double_pair_rearrange[T: AddCommGroup](a: T, b: T) {
    (a + a) + (b + b) = (a + b) + (a + b)
} by {
    a + (a + (b + b)) = a + (b + (a + b))
}

/// The two middle terms in a four-term sum may be exchanged.
theorem point2_add_pair_rearrange[T: AddCommGroup](a: T, b: T, c: T, d: T) {
    (a + b) + (c + d) = (a + c) + (b + d)
} by {
    (c + b) + d = c + (b + d)
}

/// The last term of a four-term sum may be moved next to the first.
theorem point2_add_four_last_next_to_first[T: AddCommGroup](a: T, b: T, c: T, d: T) {
    a + b + c + d = (a + d) + (b + c)
} by {
    a + b + c + d = ((a + b) + c) + d
}

/// The first, fourth, third, and sixth terms of a six-term sum may be paired.
theorem point2_add_six_pair_rearrange[T: AddCommGroup](a: T, b: T, c: T, d: T, e: T, f: T) {
    (a + b + c) + (d + e + f) = (a + d) + (c + f) + (b + e)
} by {
    point2_add_four_last_next_to_first(a, b, c, d)
    ((a + d) + ((b + c) + e)) + f = ((a + d) + (c + (b + e))) + f
    (a + d) + ((c + f) + (b + e)) = (a + d) + (c + f) + (b + e)
}

/// Subtraction through an intermediate element decomposes a difference.
theorem point2_scalar_sub_through[T: AddCommGroup](x: T, y: T, z: T) {
    (y - x) + (z - y) = z - x
} by {
    point2_add_four_last_next_to_first(y, -x, z, -y)
}

/// Reversing a scalar difference negates it.
theorem point2_scalar_sub_reverse_neg[T: AddCommGroup](x: T, y: T) {
    x - y = -(y - x)
} by {
}

/// A difference of sums may be regrouped as a sum of differences.
theorem point2_scalar_sub_pair_rearrange[T: AddCommGroup](a: T, b: T, c: T, d: T) {
    (a + b) - (c + d) = (a - c) + (b - d)
} by {
    point2_add_pair_rearrange(a, b, -c, -d)
}

/// The six terms in the two-coordinate Pythagoras expansion may be regrouped.
theorem point2_pythagoras_rearrange[T: AddCommGroup](xx: T, yy: T, uu: T, vv: T, xu: T, yv: T) {
    (xx + xu + (xu + uu)) + (yy + yv + (yv + vv)) =
    (xx + yy) + (uu + vv) + (xu + yv) + (xu + yv)
} by {
    point2_add_repeated_middle(xx, xu, uu)
    point2_add_repeated_middle(yy, yv, vv)
    point2_double_pair_rearrange(xu, yv)
    point2_add_six_pair_rearrange(xx, xu + xu, uu, yy, yv + yv, vv)
}

attributes Point2[T: CommRing] {
    /// True when two points are orthogonal as coordinate vectors.
    define orthogonal(self, other: Point2[T]) -> Bool {
        self.dot(other) = T.0
    }
}

/// Reversing a point difference negates it.
theorem point2_sub_reverse_neg[T: AddCommGroup](p: Point2[T], q: Point2[T]) {
    p.sub(q) = q.sub(p).neg
} by {
    let lhs = p.sub(q)
    let rhs = q.sub(p).neg
    point2_scalar_sub_reverse_neg(p.x, q.x)
    lhs.x = rhs.x
    point2_scalar_sub_reverse_neg(p.y, q.y)
    lhs.y = rhs.y
    point2_ext(lhs, rhs)
}

/// A point difference is addition of the additive inverse.
theorem point2_sub_eq_add_neg[T: AddCommGroup](p: Point2[T], q: Point2[T]) {
    p.sub(q) = p.add(q.neg)
} by {
    let lhs = p.sub(q)
    let rhs = p.add(q.neg)
    lhs.x = p.x - q.x
    rhs.x = p.x + -q.x
    lhs.x = rhs.x
    lhs.y = p.y - q.y
    rhs.y = p.y + -q.y
    lhs.y = rhs.y
    point2_ext(lhs, rhs)
}

/// Subtracting sums decomposes into the sum of subtractions.
theorem point2_sub_add_sub[T: AddCommGroup](a: Point2[T], b: Point2[T], c: Point2[T], d: Point2[T]) {
    a.add(b).sub(c.add(d)) = a.sub(c).add(b.sub(d))
} by {
    let lhs = a.add(b).sub(c.add(d))
    let rhs = a.sub(c).add(b.sub(d))
    lhs.x = (a.x + b.x) - (c.x + d.x)
    rhs.x = (a.x - c.x) + (b.x - d.x)
    point2_scalar_sub_pair_rearrange(a.x, b.x, c.x, d.x)
    lhs.x = rhs.x
    lhs.y = (a.y + b.y) - (c.y + d.y)
    rhs.y = (a.y - c.y) + (b.y - d.y)
    point2_scalar_sub_pair_rearrange(a.y, b.y, c.y, d.y)
    lhs.y = rhs.y
    point2_ext(lhs, rhs)
}

/// Subtraction through an intermediate point decomposes a displacement.
theorem point2_sub_through[T: AddCommGroup](a: Point2[T], b: Point2[T], c: Point2[T]) {
    b.sub(a).add(c.sub(b)) = c.sub(a)
} by {
    let lhs = b.sub(a).add(c.sub(b))
    let rhs = c.sub(a)
    lhs.x = (b.x - a.x) + (c.x - b.x)
    rhs.x = c.x - a.x
    point2_scalar_sub_through(a.x, b.x, c.x)
    lhs.x = rhs.x
    lhs.y = (b.y - a.y) + (c.y - b.y)
    rhs.y = c.y - a.y
    point2_scalar_sub_through(a.y, b.y, c.y)
    lhs.y = rhs.y
    point2_ext(lhs, rhs)
}

/// Subtracting the origin leaves a point unchanged.
theorem point2_sub_zero_right[T: AddCommGroup](p: Point2[T]) {
    p.sub(point2_zero[T]) = p
} by {
    point2_add_comm(p, point2_zero[T])
    let lhs = p.sub(point2_zero[T])
    lhs.x = p.x - T.0
    p.x - T.0 = p.x
    lhs.x = p.x
    lhs.y = p.y - T.0
    p.y - T.0 = p.y
    point2_ext(lhs, p)
}

/// The dot product with the origin on the left is zero.
theorem point2_dot_zero_left[T: CommRing](p: Point2[T]) {
    point2_zero[T].dot(p) = T.0
} by {
    point2_cross_zero_left(p)
    point2_cross_zero_right(p)
    mul_zero_left[T](p.x)
    mul_zero_left[T](p.y)
}

/// The dot product with the origin on the right is zero.
theorem point2_dot_zero_right[T: CommRing](p: Point2[T]) {
    p.dot(point2_zero[T]) = T.0
} by {
    point2_dot_zero_left(p)
}

/// Negating the left argument negates a dot product.
theorem point2_dot_neg_left[T: CommRing](p: Point2[T], q: Point2[T]) {
    p.neg.dot(q) = -p.dot(q)
} by {
    -(p.x * q.x + p.y * q.y) = -(p.x * q.x) + -(p.y * q.y)
}

/// Negating the right argument negates a dot product.
theorem point2_dot_neg_right[T: CommRing](p: Point2[T], q: Point2[T]) {
    p.dot(q.neg) = -p.dot(q)
} by {
    point2_dot_neg_left(q, p)
}

/// The dot product distributes over addition on the left.
theorem point2_dot_add_left[T: CommRing](a: Point2[T], b: Point2[T], c: Point2[T]) {
    a.add(b).dot(c) = a.dot(c) + b.dot(c)
} by {
    let x = a.x * c.x
    let y = b.x * c.x
    let z = a.y * c.y
    let w = b.y * c.y
    (a.x + b.x) * c.x = x + y
    (a.y + b.y) * c.y = z + w
    point2_add_pair_rearrange(x, y, z, w)
}

/// The dot product distributes over addition on the right.
theorem point2_dot_add_right[T: CommRing](a: Point2[T], b: Point2[T], c: Point2[T]) {
    a.dot(b.add(c)) = a.dot(b) + a.dot(c)
} by {
    point2_dot_add_left(b, c, a)
}

/// The dot product distributes over subtraction on the left.
theorem point2_dot_sub_left[T: CommRing](a: Point2[T], b: Point2[T], c: Point2[T]) {
    a.sub(b).dot(c) = a.dot(c) - b.dot(c)
} by {
    point2_sub_eq_add_neg(a, b)
    point2_dot_add_left(a, b.neg, c)
    point2_dot_neg_left(b, c)
}

/// The dot product distributes over subtraction on the right.
theorem point2_dot_sub_right[T: CommRing](a: Point2[T], b: Point2[T], c: Point2[T]) {
    a.dot(b.sub(c)) = a.dot(b) - a.dot(c)
} by {
    point2_dot_sub_left(b, c, a)
}

/// A scalar may be factored out of the left argument of a dot product.
theorem point2_dot_smul_left[T: CommRing](scalar: T, a: Point2[T], b: Point2[T]) {
    a.smul(scalar).dot(b) = scalar * a.dot(b)
} by {
    scalar * (a.x * b.x) + scalar * (a.y * b.y) = scalar * (a.x * b.x + a.y * b.y)
}

/// A scalar may be factored out of the right argument of a dot product.
theorem point2_dot_smul_right[T: CommRing](scalar: T, a: Point2[T], b: Point2[T]) {
    a.dot(b.smul(scalar)) = scalar * a.dot(b)
} by {
    point2_dot_smul_left(scalar, b, a)
}

/// The squared norm of a scalar multiple scales by the square of the scalar.
theorem point2_norm_sq_smul[T: CommRing](scalar: T, p: Point2[T]) {
    p.smul(scalar).norm_sq = scalar * scalar * p.norm_sq
} by {
    point2_dot_smul_left(scalar, p, p.smul(scalar))
    point2_dot_smul_right(scalar, p, p)
}

/// Negating the left argument negates a cross product.
theorem point2_cross_neg_left[T: CommRing](a: Point2[T], b: Point2[T]) {
    a.neg.cross(b) = -a.cross(b)
} by {
    let x = a.x * b.y
    let y = a.y * b.x
    a.neg.cross(b) = (-a.x) * b.y - (-a.y) * b.x
    (-a.x) * b.y = -x
    (-a.y) * b.x = -y
    -x - -y = y - x
    point2_scalar_sub_reverse_neg(y, x)
}

/// Negating the right argument negates a cross product.
theorem point2_cross_neg_right[T: CommRing](a: Point2[T], b: Point2[T]) {
    a.cross(b.neg) = -a.cross(b)
} by {
    let x = a.x * b.y
    let y = a.y * b.x
    a.cross(b.neg) = a.x * (-b.y) - a.y * (-b.x)
    a.x * (-b.y) = -x
    a.y * (-b.x) = -y
    -x - -y = y - x
    point2_scalar_sub_reverse_neg(y, x)
}

/// Swapping the arguments negates a cross product.
theorem point2_cross_swap[T: CommRing](a: Point2[T], b: Point2[T]) {
    a.cross(b) = -b.cross(a)
} by {
    point2_cross_self(a)
    b.cross(a) = a.y * b.x - a.x * b.y
    point2_scalar_sub_reverse_neg(a.x * b.y, a.y * b.x)
}

/// The cross product distributes over addition on the left.
theorem point2_cross_add_left[T: CommRing](a: Point2[T], b: Point2[T], c: Point2[T]) {
    a.add(b).cross(c) = a.cross(c) + b.cross(c)
} by {
    let x = a.x * c.y
    let y = b.x * c.y
    let z = a.y * c.x
    let w = b.y * c.x
    (a.x + b.x) * c.y = x + y
    (a.y + b.y) * c.x = z + w
    point2_scalar_sub_pair_rearrange(x, y, z, w)
}

/// The cross product distributes over addition on the right.
theorem point2_cross_add_right[T: CommRing](a: Point2[T], b: Point2[T], c: Point2[T]) {
    a.cross(b.add(c)) = a.cross(b) + a.cross(c)
} by {
    let x = a.x * b.y
    let y = a.x * c.y
    let z = a.y * b.x
    let w = a.y * c.x
    a.x * (b.y + c.y) = x + y
    a.y * (b.x + c.x) = z + w
    point2_scalar_sub_pair_rearrange(x, y, z, w)
}

/// A scalar may be factored out of the left argument of a cross product.
theorem point2_cross_smul_left[T: CommRing](scalar: T, a: Point2[T], b: Point2[T]) {
    a.smul(scalar).cross(b) = scalar * a.cross(b)
} by {
    let x = a.x * b.y
    let y = a.y * b.x
    (scalar * a.x) * b.y = scalar * x
    (scalar * a.y) * b.x = scalar * y
}

/// A scalar may be factored out of the right argument of a cross product.
theorem point2_cross_smul_right[T: CommRing](scalar: T, a: Point2[T], b: Point2[T]) {
    a.cross(b.smul(scalar)) = scalar * a.cross(b)
} by {
    let x = a.x * b.y
    let y = a.y * b.x
    a.x * (scalar * b.y) = scalar * x
    a.y * (scalar * b.x) = scalar * y
}

/// The cross product distributes over subtraction on the left.
theorem point2_cross_sub_left[T: CommRing](a: Point2[T], b: Point2[T], c: Point2[T]) {
    a.sub(b).cross(c) = a.cross(c) - b.cross(c)
} by {
    point2_sub_eq_add_neg(a, b)
    point2_cross_add_left(a, b.neg, c)
    point2_cross_neg_left(b, c)
}

/// The cross product distributes over subtraction on the right.
theorem point2_cross_sub_right[T: CommRing](a: Point2[T], b: Point2[T], c: Point2[T]) {
    a.cross(b.sub(c)) = a.cross(b) - a.cross(c)
} by {
    point2_sub_eq_add_neg(b, c)
    point2_cross_add_right(a, b, c.neg)
    point2_cross_neg_right(a, c)
}

/// The squared norm is nonnegative over an ordered field.
theorem point2_norm_sq_nonneg[T: OrderedField](p: Point2[T]) {
    p.norm_sq >= T.0
} by {
    T.0 <= p.x * p.x
    T.0 <= p.y * p.y
    T.0 + T.0 <= p.x * p.x + p.y * p.y
    p.norm_sq = p.x * p.x + p.y * p.y
}

/// The origin has zero squared norm.
theorem point2_norm_sq_zero[T: CommRing](p: Point2[T]) {
    p = point2_zero[T] implies p.norm_sq = T.0
} by {
    if p = point2_zero[T] {
        point2_dot_zero_left(p)
    }
}

/// Negation preserves squared norm.
theorem point2_norm_sq_neg[T: CommRing](p: Point2[T]) {
    p.neg.norm_sq = p.norm_sq
} by {
    point2_dot_neg_left(p, p.neg)
    point2_dot_neg_right(p, p)
    --p.dot(p) = p.dot(p)
}

/// The squared norm of a sum expands by the dot product.
theorem point2_norm_sq_add_expansion[T: CommRing](a: Point2[T], b: Point2[T]) {
    a.add(b).norm_sq = a.norm_sq + b.norm_sq + a.dot(b) + a.dot(b)
} by {
    a.add(b).dot(a.add(b)) =
        a.add(b).x * a.add(b).x + a.add(b).y * a.add(b).y
    point2_ring_square_add(a.x, b.x)
    point2_ring_square_add(a.y, b.y)
    point2_pythagoras_rearrange(
        a.x * a.x,
        a.y * a.y,
        b.x * b.x,
        b.y * b.y,
        a.x * b.x,
        a.y * b.y)
}

/// The squared norm of a difference expands by the dot product.
theorem point2_norm_sq_sub_expansion[T: CommRing](a: Point2[T], b: Point2[T]) {
    a.sub(b).norm_sq = a.norm_sq + b.norm_sq - a.dot(b) - a.dot(b)
} by {
    point2_sub_eq_add_neg(a, b)
    point2_norm_sq_add_expansion(a, b.neg)
    point2_norm_sq_neg(b)
    point2_dot_neg_right(a, b)
}

/// The squared distance from a point to itself is zero.
theorem point2_dist_sq_self[T: CommRing](p: Point2[T]) {
    p.dist_sq(p) = T.0
} by {
    point2_sub_self(p)
}

/// Squared distance is symmetric.
theorem point2_dist_sq_comm[T: CommRing](p: Point2[T], q: Point2[T]) {
    p.dist_sq(q) = q.dist_sq(p)
} by {
    point2_sub_reverse_neg(p, q)
    point2_norm_sq_neg(q.sub(p))
}

/// Squared distance is the squared norm of a difference.
theorem point2_dist_sq_eq_norm_sq_sub[T: CommRing](p: Point2[T], q: Point2[T]) {
    p.dist_sq(q) = p.sub(q).norm_sq
}

/// Squared distance to the origin is the squared norm.
theorem point2_dist_sq_zero_right[T: CommRing](p: Point2[T]) {
    p.dist_sq(point2_zero[T]) = p.norm_sq
} by {
    point2_sub_zero_right(p)
}

/// Orthogonality is symmetric.
theorem point2_orthogonal_comm[T: CommRing](p: Point2[T], q: Point2[T]) {
    p.orthogonal(q) = q.orthogonal(p)
} by {
    point2_dot_comm(p, q)
}

/// Negating the left argument preserves orthogonality.
theorem point2_orthogonal_neg_left[T: CommRing](p: Point2[T], q: Point2[T]) {
    p.neg.orthogonal(q) = p.orthogonal(q)
} by {
    if p.neg.orthogonal(q) {
        point2_dot_neg_left(p, q)
        -p.dot(q) = T.0
        p.dot(q) = T.0
        p.orthogonal(q)
    }
    if p.orthogonal(q) {
        point2_dot_neg_left(p, q)
        -p.dot(q) = -T.0
        -T.0 = T.0
        p.neg.orthogonal(q)
    }
}

/// Negating the right argument preserves orthogonality.
theorem point2_orthogonal_neg_right[T: CommRing](p: Point2[T], q: Point2[T]) {
    p.orthogonal(q.neg) = p.orthogonal(q)
} by {
    q.orthogonal(p) = p.orthogonal(q)
}

/// Scaling the left argument by a scalar preserves orthogonality from an orthogonal pair.
theorem point2_orthogonal_smul_left[T: CommRing](scalar: T, p: Point2[T], q: Point2[T]) {
    p.orthogonal(q) implies p.smul(scalar).orthogonal(q)
} by {
    if p.orthogonal(q) {
        p.dot(q) = T.0
        point2_dot_smul_left(scalar, p, q)
        mul_zero_right[T](scalar)
        scalar * p.dot(q) = T.0
    }
}

/// Scaling the right argument by a scalar preserves orthogonality from an orthogonal pair.
theorem point2_orthogonal_smul_right[T: CommRing](scalar: T, p: Point2[T], q: Point2[T]) {
    p.orthogonal(q) implies p.orthogonal(q.smul(scalar))
} by {
    if p.orthogonal(q) {
        p.dot(q) = T.0
        point2_dot_smul_right(scalar, p, q)
        mul_zero_right[T](scalar)
        scalar * p.dot(q) = T.0
    }
}

/// Pythagoras for orthogonal coordinate vectors.
theorem point2_pythagoras_vectors[T: CommRing](p: Point2[T], q: Point2[T]) {
    p.orthogonal(q) implies p.add(q).norm_sq = p.norm_sq + q.norm_sq
} by {
    if p.orthogonal(q) {
        p.dot(q) = T.0
        point2_norm_sq_add_expansion(p, q)
        p.add(q).norm_sq = p.norm_sq + q.norm_sq
    }
}

/// Pythagoras for three points using the two successive displacements.
theorem point2_pythagoras_points_from_ab[T: CommRing](a: Point2[T], b: Point2[T], c: Point2[T]) {
    b.sub(a).orthogonal(c.sub(b)) implies
    c.sub(a).norm_sq = b.sub(a).norm_sq + c.sub(b).norm_sq
} by {
    if b.sub(a).orthogonal(c.sub(b)) {
        point2_sub_through(a, b, c)
        c.sub(a).norm_sq = b.sub(a).add(c.sub(b)).norm_sq
        point2_pythagoras_vectors(b.sub(a), c.sub(b))
    }
}

/// Pythagoras for three points with the first displacement reversed.
theorem point2_pythagoras_points[T: CommRing](a: Point2[T], b: Point2[T], c: Point2[T]) {
    a.sub(b).orthogonal(c.sub(b)) implies
    c.sub(a).norm_sq = b.sub(a).norm_sq + c.sub(b).norm_sq
} by {
    if a.sub(b).orthogonal(c.sub(b)) {
        point2_sub_reverse_neg(a, b)
        point2_orthogonal_neg_left(b.sub(a), c.sub(b))
        b.sub(a).orthogonal(c.sub(b))
        point2_pythagoras_points_from_ab(a, b, c)
    }
}
