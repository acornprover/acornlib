from ordered_field import OrderedField
from list import List
from geometry.point2 import Point2
from geometry.point2_segment import point2_on_segment_start,
    point2_on_segment_end, point2_on_segment_translate,
    point2_on_segment_translate_forward
from geometry.point2_polygon import point2_translate_points,
    point2_translate_points_cons

/// True when a point lies on one of the consecutive segments of a chain.
define point2_polyline_chain_contains[T: OrderedField](previous: Point2[T], rest: List[Point2[T]], p: Point2[T]) -> Bool {
    match rest {
        List.nil {
            false
        }
        List.cons(next, tail) {
            if previous.on_segment(next, p) {
                true
            } else {
                point2_polyline_chain_contains(next, tail, p)
            }
        }
    }
}

/// True when a point lies on one of the consecutive segments of a point list.
define point2_polyline_contains[T: OrderedField](points: List[Point2[T]], p: Point2[T]) -> Bool {
    match points {
        List.nil {
            false
        }
        List.cons(first, tail) {
            match tail {
                List.nil {
                    false
                }
                List.cons(second, rest) {
                    point2_polyline_chain_contains(first, List.cons(second, rest), p)
                }
            }
        }
    }
}

/// An empty edge chain contains no point.
theorem point2_polyline_chain_contains_nil[T: OrderedField](previous: Point2[T], p: Point2[T]) {
    not point2_polyline_chain_contains(previous, List.nil[Point2[T]], p)
}

/// A one-edge chain contains exactly the points on that segment.
theorem point2_polyline_chain_contains_single[T: OrderedField](previous: Point2[T], next: Point2[T], p: Point2[T]) {
    point2_polyline_chain_contains(previous, List.cons(next, List.nil[Point2[T]]), p) =
    previous.on_segment(next, p)
} by {
    if previous.on_segment(next, p) {
        point2_polyline_chain_contains(previous, List.cons(next, List.nil[Point2[T]]), p)
    } else {
        not point2_polyline_chain_contains(previous, List.cons(next, List.nil[Point2[T]]), p)
    }
}

/// The first endpoint of the first edge lies in a nonempty edge chain.
theorem point2_polyline_chain_contains_start[T: OrderedField](previous: Point2[T], next: Point2[T], tail: List[Point2[T]]) {
    point2_polyline_chain_contains(previous, List.cons(next, tail), previous)
} by {
    point2_on_segment_start(previous, next)
}

/// The second endpoint of the first edge lies in a nonempty edge chain.
theorem point2_polyline_chain_contains_next[T: OrderedField](previous: Point2[T], next: Point2[T], tail: List[Point2[T]]) {
    point2_polyline_chain_contains(previous, List.cons(next, tail), next)
} by {
    point2_on_segment_end(previous, next)
}

/// If a point lies on the first segment, it lies in the whole edge chain.
theorem point2_polyline_chain_contains_first_edge[T: OrderedField](previous: Point2[T], next: Point2[T], tail: List[Point2[T]], p: Point2[T]) {
    previous.on_segment(next, p) implies point2_polyline_chain_contains(previous, List.cons(next, tail), p)
} by {
    if previous.on_segment(next, p) {
        point2_polyline_chain_contains(previous, List.cons(next, tail), p)
    }
}

/// If a point lies in the tail chain and not on the first segment, it lies in the whole edge chain.
theorem point2_polyline_chain_contains_tail[T: OrderedField](previous: Point2[T], next: Point2[T], tail: List[Point2[T]], p: Point2[T]) {
    not previous.on_segment(next, p) and point2_polyline_chain_contains(next, tail, p) implies
    point2_polyline_chain_contains(previous, List.cons(next, tail), p)
} by {
    if not previous.on_segment(next, p) and point2_polyline_chain_contains(next, tail, p) {
        point2_polyline_chain_contains(previous, List.cons(next, tail), p)
    }
}

/// The empty point list contains no polyline segment point.
theorem point2_polyline_contains_nil[T: OrderedField](p: Point2[T]) {
    not point2_polyline_contains(List.nil[Point2[T]], p)
}

/// A singleton point list contains no polyline segment point.
theorem point2_polyline_contains_single_point[T: OrderedField](a: Point2[T], p: Point2[T]) {
    not point2_polyline_contains(List.cons(a, List.nil[Point2[T]]), p)
}

/// A two-point list contains exactly the points on its single segment.
theorem point2_polyline_contains_pair[T: OrderedField](a: Point2[T], b: Point2[T], p: Point2[T]) {
    point2_polyline_contains(List.cons(a, List.cons(b, List.nil[Point2[T]])), p) = a.on_segment(b, p)
} by {
    point2_polyline_chain_contains_single(a, b, p)
}

/// The first endpoint of a two-point list lies on the polyline.
theorem point2_polyline_contains_pair_start[T: OrderedField](a: Point2[T], b: Point2[T]) {
    point2_polyline_contains(List.cons(a, List.cons(b, List.nil[Point2[T]])), a)
} by {
    point2_on_segment_start(a, b)
}

/// The second endpoint of a two-point list lies on the polyline.
theorem point2_polyline_contains_pair_end[T: OrderedField](a: Point2[T], b: Point2[T]) {
    point2_polyline_contains(List.cons(a, List.cons(b, List.nil[Point2[T]])), b)
} by {
    point2_on_segment_end(a, b)
}

/// The first point of a list with at least two points lies on its polyline.
theorem point2_polyline_contains_first[T: OrderedField](a: Point2[T], b: Point2[T], rest: List[Point2[T]]) {
    point2_polyline_contains(List.cons(a, List.cons(b, rest)), a)
} by {
    point2_polyline_chain_contains_start(a, b, rest)
}

/// The second point of a list with at least two points lies on its polyline.
theorem point2_polyline_contains_second[T: OrderedField](a: Point2[T], b: Point2[T], rest: List[Point2[T]]) {
    point2_polyline_contains(List.cons(a, List.cons(b, rest)), b)
} by {
    point2_polyline_chain_contains_next(a, b, rest)
}

/// Translating a point in an edge chain gives a point in the translated edge chain.
theorem point2_polyline_chain_contains_translate_forward[T: OrderedField](
    previous: Point2[T],
    rest: List[Point2[T]],
    p: Point2[T],
    v: Point2[T]
) {
    point2_polyline_chain_contains(previous, rest, p) implies
    point2_polyline_chain_contains(previous.translate(v), point2_translate_points(rest, v), p.translate(v))
} by {
    define h(xs: List[Point2[T]], prev: Point2[T]) -> Bool {
        point2_polyline_chain_contains(prev, xs, p) implies
        point2_polyline_chain_contains(prev.translate(v), point2_translate_points(xs, v), p.translate(v))
    }
    define q(xs: List[Point2[T]]) -> Bool {
        forall(prev: Point2[T]) { h(xs, prev) }
    }

    forall(prev: Point2[T]) {
        h(List.nil[Point2[T]], prev)
    }
    q(List.nil[Point2[T]])

    forall(head: Point2[T], tail: List[Point2[T]]) {
        if q(tail) {
            forall(prev: Point2[T]) {
                if point2_polyline_chain_contains(prev, List.cons(head, tail), p) {
                    point2_translate_points_cons(head, tail, v)
                    point2_on_segment_translate(prev, head, p, v)
                    if prev.on_segment(head, p) {
                        point2_on_segment_translate_forward(prev, head, p, v)
                        point2_polyline_chain_contains_first_edge(prev.translate(v), head.translate(v), point2_translate_points(tail, v), p.translate(v))
                        point2_polyline_chain_contains(prev.translate(v), point2_translate_points(List.cons(head, tail), v), p.translate(v))
                    } else {
                        h(tail, head)
                        point2_polyline_chain_contains(head.translate(v), point2_translate_points(tail, v), p.translate(v))
                        point2_polyline_chain_contains_tail(prev.translate(v), head.translate(v), point2_translate_points(tail, v), p.translate(v))
                        point2_polyline_chain_contains(prev.translate(v), point2_translate_points(List.cons(head, tail), v), p.translate(v))
                    }
                    h(List.cons(head, tail), prev)
                } else {
                    h(List.cons(head, tail), prev)
                }
                h(List.cons(head, tail), prev)
            }
            q(List.cons(head, tail))
        }
    }

    List.induction(q)
    q(rest)
    h(rest, previous)
}

/// Translating a point on a polyline gives a point on the translated polyline.
theorem point2_polyline_contains_translate_forward[T: OrderedField](points: List[Point2[T]], p: Point2[T], v: Point2[T]) {
    point2_polyline_contains(points, p) implies point2_polyline_contains(point2_translate_points(points, v), p.translate(v))
} by {
    if point2_polyline_contains(points, p) {
        match points {
            List.nil {
            }
            List.cons(first, tail) {
                match tail {
                    List.nil {
                        false
                    }
                    List.cons(second, rest) {
                        point2_translate_points_cons(first, tail, v)
                        point2_translate_points_cons(second, rest, v)
                        point2_polyline_chain_contains_translate_forward(first, List.cons(second, rest), p, v)
                        point2_polyline_chain_contains(first.translate(v), point2_translate_points(List.cons(second, rest), v), p.translate(v))
                        point2_polyline_contains(List.cons(first.translate(v), List.cons(second.translate(v), point2_translate_points(rest, v))), p.translate(v))
                        point2_polyline_contains(point2_translate_points(points, v), p.translate(v))
                    }
                }
            }
        }
    }
}
