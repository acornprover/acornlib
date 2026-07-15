/// Polygon-list area behavior under append, reversal, and small cyclic moves.

from ordered_field import OrderedField
from list import List, reverse
from geometry.point2 import Point2
from geometry.point2_triangle import point2_triangle_area2_eq_orientation,
    point2_triangle_area2_rotate, point2_triangle_area2_swap_first_third_neg
from geometry.point2_algebra import point2_sub_through, point2_cross_add_left,
    point2_cross_add_right, point2_cross_self
from geometry.point2_polygon import point2_translate_points,
    point2_polygon_chain_area2, point2_polygon_area2,
    point2_polygon_area2_pair, point2_polygon_area2_triple, point2_polygon_area2_quad

/// Translating a singleton point list translates its element.
theorem point2_translate_points_singleton[T: OrderedField](p: Point2[T], v: Point2[T]) {
    point2_translate_points(List.singleton(p), v) = List.singleton(p.translate(v))
} by {
    point2_translate_points(List.nil[Point2[T]], v) = List.nil[Point2[T]]
}

/// Translating a two-point list translates both elements.
theorem point2_translate_points_pair[T: OrderedField](a: Point2[T], b: Point2[T], v: Point2[T]) {
    point2_translate_points(List.cons(a, List.cons(b, List.nil[Point2[T]])), v) =
    List.cons(a.translate(v), List.cons(b.translate(v), List.nil[Point2[T]]))
} by {
    point2_translate_points(List.cons(b, List.nil[Point2[T]]), v) =
        List.cons(b.translate(v), point2_translate_points(List.nil[Point2[T]], v))
}

/// Translating commutes with appending to an empty point list.
theorem point2_translate_points_append_nil[T: OrderedField](last: Point2[T], v: Point2[T]) {
    point2_translate_points(List.nil[Point2[T]].append(last), v) =
    point2_translate_points(List.nil[Point2[T]], v).append(last.translate(v))
} by {
    point2_translate_points_singleton(last, v)
}

/// Translating commutes with appending to a singleton point list.
theorem point2_translate_points_append_singleton[T: OrderedField](a: Point2[T], last: Point2[T], v: Point2[T]) {
    point2_translate_points(List.singleton(a).append(last), v) =
    point2_translate_points(List.singleton(a), v).append(last.translate(v))
} by {
    List.cons(a, List.nil[Point2[T]]) + List.singleton(last) =
        List.cons(a, List.nil[Point2[T]] + List.singleton(last))
    point2_translate_points(List.singleton(a).append(last), v) =
        point2_translate_points(List.cons(a, List.cons(last, List.nil[Point2[T]])), v)
    point2_translate_points_pair(a, last, v)
    point2_translate_points_singleton(a, v)
    List.cons(a.translate(v), List.nil[Point2[T]]) + List.singleton(last.translate(v)) =
        List.cons(a.translate(v), List.nil[Point2[T]] + List.singleton(last.translate(v)))
    point2_translate_points(List.singleton(a), v).append(last.translate(v)) =
        List.cons(a.translate(v), List.cons(last.translate(v), List.nil[Point2[T]]))
}

/// The final point of a list, using `default` for the empty list.
define point2_list_last_or[T: OrderedField](default: Point2[T], xs: List[Point2[T]]) -> Point2[T] {
    match xs {
        List.nil {
            default
        }
        List.cons(head, tail) {
            point2_list_last_or(head, tail)
        }
    }
}

/// The last-or-default value for an empty list is the default.
theorem point2_list_last_or_nil[T: OrderedField](default: Point2[T]) {
    point2_list_last_or(default, List.nil[Point2[T]]) = default
}

/// The last-or-default value for a cons list recurses with the head as default.
theorem point2_list_last_or_cons[T: OrderedField](default: Point2[T], head: Point2[T], tail: List[Point2[T]]) {
    point2_list_last_or(default, List.cons(head, tail)) = point2_list_last_or(head, tail)
}

/// Appending one final point to an empty chain gives exactly the final triangle from the previous point.
theorem point2_polygon_chain_area2_append_single_nil[T: OrderedField](
    anchor: Point2[T], previous: Point2[T], last: Point2[T]
) {
    point2_polygon_chain_area2(anchor, previous, List.nil[Point2[T]].append(last)) =
    point2_polygon_chain_area2(anchor, previous, List.nil[Point2[T]]) +
    anchor.triangle_area2(point2_list_last_or(previous, List.nil[Point2[T]]), last)
} by {
    point2_polygon_chain_area2(anchor, previous, List.singleton(last)) = anchor.triangle_area2(previous, last)
}

/// Appending one final point to a singleton chain splits off the final fan triangle.
theorem point2_polygon_chain_area2_append_singleton[T: OrderedField](
    anchor: Point2[T], previous: Point2[T], only: Point2[T], last: Point2[T]
) {
    point2_polygon_chain_area2(anchor, previous, List.singleton(only).append(last)) =
    point2_polygon_chain_area2(anchor, previous, List.singleton(only)) +
    anchor.triangle_area2(point2_list_last_or(previous, List.singleton(only)), last)
} by {
    List.cons(only, List.nil[Point2[T]]) + List.singleton(last) =
        List.cons(only, List.nil[Point2[T]] + List.singleton(last))
    List.nil[Point2[T]].append(last) = List.singleton(last)
    point2_polygon_chain_area2(anchor, previous, List.singleton(only).append(last)) =
        anchor.triangle_area2(previous, only) + point2_polygon_chain_area2(anchor, only, List.singleton(last))
    point2_list_last_or(previous, List.singleton(only)) = only
}

/// Triangle doubled area as a cross product of consecutive edge displacements.
theorem point2_triangle_area2_chain_cross[T: OrderedField](a: Point2[T], b: Point2[T], c: Point2[T]) {
    a.triangle_area2(b, c) = b.sub(a).cross(c.sub(b))
} by {
    let u = b.sub(a)
    let v = c.sub(b)
    point2_triangle_area2_eq_orientation(a, b, c)
    point2_sub_through(a, b, c)
    point2_cross_add_right(u, u, v)
    point2_cross_self(u)
}

/// Splitting a quadrilateral fan along the other diagonal preserves doubled area.
theorem point2_quad_fan_area2_split[T: OrderedField](a: Point2[T], b: Point2[T], c: Point2[T], d: Point2[T]) {
    a.triangle_area2(b, c) + a.triangle_area2(c, d) =
    a.triangle_area2(b, d) + b.triangle_area2(c, d)
} by {
    let u = b.sub(a)
    let v = c.sub(b)
    let w = d.sub(c)
    point2_triangle_area2_chain_cross(a, b, c)
    point2_triangle_area2_chain_cross(b, c, d)
    point2_triangle_area2_chain_cross(a, c, d)
    point2_sub_through(a, b, c)
    point2_cross_add_left(u, v, w)
    point2_triangle_area2_chain_cross(a, b, d)
    point2_sub_through(b, c, d)
    point2_cross_add_right(u, v, w)
    u.cross(v) + (u.cross(w) + v.cross(w)) = (u.cross(v) + u.cross(w)) + v.cross(w)
}

/// Cyclically moving the first vertex of a quadrilateral to the end preserves doubled area.
theorem point2_polygon_area2_quad_cyclic_shift[T: OrderedField](a: Point2[T], b: Point2[T], c: Point2[T], d: Point2[T]) {
    point2_polygon_area2(List.cons(b, List.cons(c, List.cons(d, List.cons(a, List.nil[Point2[T]]))))) =
    point2_polygon_area2(List.cons(a, List.cons(b, List.cons(c, List.cons(d, List.nil[Point2[T]])))))
} by {
    point2_polygon_area2_quad(b, c, d, a)
    point2_polygon_area2_quad(a, b, c, d)
    point2_quad_fan_area2_split(a, b, c, d)
    point2_triangle_area2_rotate(a, b, d)
    b.triangle_area2(d, a) + b.triangle_area2(c, d) =
        b.triangle_area2(c, d) + b.triangle_area2(d, a)
}

/// Reversing a three-point list gives the explicitly reversed three-point list.
theorem point2_reverse_triple_list[T: OrderedField](a: Point2[T], b: Point2[T], c: Point2[T]) {
    reverse(List.cons(a, List.cons(b, List.cons(c, List.nil[Point2[T]])))) =
    List.cons(c, List.cons(b, List.cons(a, List.nil[Point2[T]])))
} by {
    List.nil[Point2[T]].append(c) = List.singleton(c)
    reverse(List.cons(c, List.nil[Point2[T]])).append(b) =
        List.cons(c, List.nil[Point2[T]]).append(b)
    List.cons(c, List.nil[Point2[T]]).append(b) =
        List.cons(c, List.nil[Point2[T]]) + List.singleton(b)
    List.cons(c, List.nil[Point2[T]]) + List.singleton(b) =
        List.cons(c, List.nil[Point2[T]] + List.singleton(b))
    List.cons(c, List.cons(b, List.nil[Point2[T]])).append(a) =
        List.cons(c, List.cons(b, List.nil[Point2[T]])) + List.singleton(a)
    List.cons(c, List.cons(b, List.nil[Point2[T]])) + List.singleton(a) =
        List.cons(c, List.cons(b, List.nil[Point2[T]]) + List.singleton(a))
    List.cons(b, List.nil[Point2[T]]) + List.singleton(a) =
        List.cons(b, List.nil[Point2[T]] + List.singleton(a))
}

/// Reversing a four-point list gives the explicitly reversed four-point list.
theorem point2_reverse_quad_list[T: OrderedField](a: Point2[T], b: Point2[T], c: Point2[T], d: Point2[T]) {
    reverse(List.cons(a, List.cons(b, List.cons(c, List.cons(d, List.nil[Point2[T]]))))) =
    List.cons(d, List.cons(c, List.cons(b, List.cons(a, List.nil[Point2[T]]))))
} by {
    point2_reverse_triple_list(b, c, d)
    List.cons(d, List.cons(c, List.cons(b, List.nil[Point2[T]]))).append(a) =
        List.cons(d, List.cons(c, List.cons(b, List.nil[Point2[T]]))) + List.singleton(a)
    List.cons(d, List.cons(c, List.cons(b, List.nil[Point2[T]]))) + List.singleton(a) =
        List.cons(d, List.cons(c, List.cons(b, List.nil[Point2[T]]) + List.singleton(a)))
    List.cons(b, List.nil[Point2[T]]) + List.singleton(a) =
        List.cons(b, List.nil[Point2[T]] + List.singleton(a))
}

/// Reversing a triangle negates its polygon signed doubled area.
theorem point2_polygon_area2_reverse_triple_neg[T: OrderedField](a: Point2[T], b: Point2[T], c: Point2[T]) {
    point2_polygon_area2(reverse(List.cons(a, List.cons(b, List.cons(c, List.nil[Point2[T]]))))) =
    -point2_polygon_area2(List.cons(a, List.cons(b, List.cons(c, List.nil[Point2[T]]))))
} by {
    point2_reverse_triple_list(a, b, c)
    point2_polygon_area2_triple(c, b, a)
    point2_polygon_area2_triple(a, b, c)
    point2_triangle_area2_swap_first_third_neg(c, b, a)
}

/// Reversing a quadrilateral negates its polygon signed doubled area.
theorem point2_polygon_area2_reverse_quad_neg[T: OrderedField](a: Point2[T], b: Point2[T], c: Point2[T], d: Point2[T]) {
    point2_polygon_area2(reverse(List.cons(a, List.cons(b, List.cons(c, List.cons(d, List.nil[Point2[T]])))))) =
    -point2_polygon_area2(List.cons(a, List.cons(b, List.cons(c, List.cons(d, List.nil[Point2[T]])))))
} by {
    point2_reverse_quad_list(a, b, c, d)
    point2_polygon_area2_quad(d, c, b, a)
    point2_polygon_area2_quad(a, b, c, d)
    point2_quad_fan_area2_split(d, c, b, a)
    point2_triangle_area2_swap_first_third_neg(a, c, d)
    point2_triangle_area2_swap_first_third_neg(a, b, c)
    -a.triangle_area2(c, d) + -a.triangle_area2(b, c) =
        -(a.triangle_area2(b, c) + a.triangle_area2(c, d))
}

/// Extending a two-point polygon by one point adds exactly the fan triangle.
theorem point2_polygon_area2_append_to_pair[T: OrderedField](a: Point2[T], b: Point2[T], c: Point2[T]) {
    point2_polygon_area2(List.cons(a, List.cons(b, List.nil[Point2[T]])).append(c)) =
    point2_polygon_area2(List.cons(a, List.cons(b, List.nil[Point2[T]]))) + a.triangle_area2(b, c)
} by {
    List.cons(a, List.cons(b, List.nil[Point2[T]])) + List.singleton(c) =
        List.cons(a, List.cons(b, List.nil[Point2[T]]) + List.singleton(c))
    point2_polygon_area2_triple(a, b, c)
    point2_polygon_area2_pair(a, b)
}

/// Extending a three-point polygon by one point adds the final fan triangle.
theorem point2_polygon_area2_append_to_triple[T: OrderedField](a: Point2[T], b: Point2[T], c: Point2[T], d: Point2[T]) {
    point2_polygon_area2(List.cons(a, List.cons(b, List.cons(c, List.nil[Point2[T]]))).append(d)) =
    point2_polygon_area2(List.cons(a, List.cons(b, List.cons(c, List.nil[Point2[T]])))) + a.triangle_area2(c, d)
} by {
    List.cons(a, List.cons(b, List.cons(c, List.nil[Point2[T]]))) + List.singleton(d) =
        List.cons(a, List.cons(b, List.cons(c, List.nil[Point2[T]]) + List.singleton(d)))
    List.cons(c, List.nil[Point2[T]]) + List.singleton(d) =
        List.cons(c, List.nil[Point2[T]] + List.singleton(d))
    point2_polygon_area2_quad(a, b, c, d)
    point2_polygon_area2_triple(a, b, c)
}
