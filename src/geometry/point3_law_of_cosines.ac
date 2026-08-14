from comm_ring import CommRing
from algebra.add_comm_group import AddCommGroup
from geometry.point3 import Point3
from geometry.point3_algebra import point3_dot_comm, point3_dist_sq_comm,
    point3_norm_sq_sub_expansion, point3_sub_same_base,
    point3_scalar_sub_pair_rearrange

/// Subtracting two terms successively is subtracting their sum.
theorem point3_sub_sub_eq_sub_add[T: AddCommGroup](x: T, y: T, z: T) {
    x - y - z = x - (y + z)
} by {
    x - y - z = x + -y + -z
    -z + -y = -y + -z
}

/// A difference from a difference can be rearranged around the removed middle term.
theorem point3_sub_sub_rearrange[T: AddCommGroup](x: T, y: T, z: T) {
    x - (y - z) = z - (y - x)
} by {
    x - (y - z) = x + -(y + -z)
    -(y + -z) = z + -y
    x + (z + -y) = z + (x + -y)
    -(y + -x) = x + -y
}

/// If one term is a difference from a sum, subtracting it recovers the removed term.
theorem point3_sub_eq_sub_imp_sub_eq[T: AddCommGroup](sum: T, removed: T, x: T) {
    x = sum - removed implies sum - x = removed
} by {
    if x = sum - removed {
        point3_sub_sub_rearrange(sum, sum, removed)
        sum - sum = T.0
        sum - x = removed
    }
}

/// Law of cosines in squared-distance / dot-product form at vertex `a`.
theorem point3_law_of_cosines_dist_sq[T: CommRing](a: Point3[T], b: Point3[T], c: Point3[T]) {
    b.dist_sq(c) =
        a.dist_sq(b) + a.dist_sq(c) -
        b.sub(a).dot(c.sub(a)) - b.sub(a).dot(c.sub(a))
} by {
    let u = b.sub(a)
    let v = c.sub(a)

    point3_sub_same_base(a, b, c)
    v.sub(u) = c.sub(b)

    point3_dist_sq_comm(b, c)
    b.dist_sq(c) = c.dist_sq(b)
    c.dist_sq(b) = c.sub(b).norm_sq
    c.sub(b).norm_sq = v.sub(u).norm_sq
    b.dist_sq(c) = v.sub(u).norm_sq

    point3_norm_sq_sub_expansion(v, u)
    v.sub(u).norm_sq = v.norm_sq + u.norm_sq - v.dot(u) - v.dot(u)

    point3_dot_comm(v, u)
    v.dot(u) = u.dot(v)
    v.norm_sq + u.norm_sq - v.dot(u) - v.dot(u) =
        u.norm_sq + v.norm_sq - u.dot(v) - u.dot(v)
    b.dist_sq(c) = u.norm_sq + v.norm_sq - u.dot(v) - u.dot(v)

    point3_dist_sq_comm(a, b)
    a.dist_sq(b) = b.dist_sq(a)
    b.dist_sq(a) = b.sub(a).norm_sq
    a.dist_sq(b) = u.norm_sq

    point3_dist_sq_comm(a, c)
    a.dist_sq(c) = c.dist_sq(a)
    c.dist_sq(a) = c.sub(a).norm_sq
    a.dist_sq(c) = v.norm_sq

    u.dot(v) = b.sub(a).dot(c.sub(a))
    b.dist_sq(c) =
        a.dist_sq(b) + a.dist_sq(c) -
        b.sub(a).dot(c.sub(a)) - b.sub(a).dot(c.sub(a))
}

/// Twice the dot product at vertex `a` is determined by the three squared side lengths.
theorem point3_double_dot_from_dist_sq[T: CommRing](a: Point3[T], b: Point3[T], c: Point3[T]) {
    b.sub(a).dot(c.sub(a)) + b.sub(a).dot(c.sub(a)) =
        a.dist_sq(b) + a.dist_sq(c) - b.dist_sq(c)
} by {
    let d = b.sub(a).dot(c.sub(a))
    let sum_sq = a.dist_sq(b) + a.dist_sq(c)

    point3_law_of_cosines_dist_sq(a, b, c)
    b.dist_sq(c) = sum_sq - d - d

    point3_sub_sub_eq_sub_add(sum_sq, d, d)
    sum_sq - d - d = sum_sq - (d + d)
    b.dist_sq(c) = sum_sq - (d + d)

    point3_sub_eq_sub_imp_sub_eq(sum_sq, d + d, b.dist_sq(c))
    sum_sq - b.dist_sq(c) = d + d
    d + d = sum_sq - b.dist_sq(c)
    b.sub(a).dot(c.sub(a)) + b.sub(a).dot(c.sub(a)) =
        a.dist_sq(b) + a.dist_sq(c) - b.dist_sq(c)
}
