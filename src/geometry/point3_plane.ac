from comm_ring import CommRing
from ordered_field import OrderedField
from geometry.point3 import Point3
from geometry.point3_algebra import point3_sub_same_base,
    point3_dot_sub_right, point3_scalar_sub_same_base

attributes Point3[T: CommRing] {
    /// True when `x` lies on the plane through `self` with normal `n`.
    define on_plane(self, n: Point3[T], x: Point3[T]) -> Bool {
        n.dot(x.sub(self)) = T.0
    }

    /// True when `p` lies on the sphere with this center and squared radius.
    define on_sphere(self, radius_sq: T, p: Point3[T]) -> Bool {
        p.dist_sq(self) = radius_sq
    }
}

/// Plane membership is equality of the normal dot product to zero.
theorem point3_on_plane_eq_dot[T: CommRing](p: Point3[T], n: Point3[T], x: Point3[T]) {
    p.on_plane(n, x) = (n.dot(x.sub(p)) = T.0)
}

/// Sphere membership is equality of squared distance to the center.
theorem point3_on_sphere_eq_dist_sq[T: CommRing](center: Point3[T], radius_sq: T, p: Point3[T]) {
    center.on_sphere(radius_sq, p) = (p.dist_sq(center) = radius_sq)
}

/// The squared distance from a point to a plane with normal `n`.
define point3_dist_to_plane_sq[T: OrderedField](p: Point3[T], n: Point3[T], x: Point3[T]) -> T {
    n.dot(x.sub(p)) * n.dot(x.sub(p)) * n.norm_sq.inverse
}

/// The displacement between two plane points is orthogonal to the normal.
theorem point3_plane_normal_orthogonal_chord[T: CommRing](
    p: Point3[T], n: Point3[T], x: Point3[T], y: Point3[T]
) {
    p.on_plane(n, x) and p.on_plane(n, y) implies n.dot(x.sub(y)) = T.0
} by {
    if p.on_plane(n, x) and p.on_plane(n, y) {
        point3_on_plane_eq_dot(p, n, x)
        p.on_plane(n, x) = (n.dot(x.sub(p)) = T.0)
        n.dot(x.sub(p)) = T.0
        point3_on_plane_eq_dot(p, n, y)
        p.on_plane(n, y) = (n.dot(y.sub(p)) = T.0)
        n.dot(y.sub(p)) = T.0
        point3_sub_same_base(p, y, x)
        x.sub(p).sub(y.sub(p)) = x.sub(y)
        point3_dot_sub_right(n, x.sub(p), y.sub(p))
        n.dot(x.sub(p).sub(y.sub(p))) = n.dot(x.sub(p)) - n.dot(y.sub(p))
        n.dot(x.sub(p)) - n.dot(y.sub(p)) = T.0
        n.dot(x.sub(p).sub(y.sub(p))) = T.0
        n.dot(x.sub(y)) = T.0
    }
}

/// The squared distance from a plane point to the plane is zero.
theorem point3_dist_to_plane_sq_of_plane_point[T: OrderedField](
    p: Point3[T], n: Point3[T], x: Point3[T]
) {
    p.on_plane(n, x) implies point3_dist_to_plane_sq(p, n, x) = T.0
} by {
    if p.on_plane(n, x) {
        point3_on_plane_eq_dot(p, n, x)
        p.on_plane(n, x) = (n.dot(x.sub(p)) = T.0)
        n.dot(x.sub(p)) = T.0
        point3_dist_to_plane_sq(p, n, x) =
            n.dot(x.sub(p)) * n.dot(x.sub(p)) * n.norm_sq.inverse
        n.dot(x.sub(p)) * n.dot(x.sub(p)) = T.0
        point3_dist_to_plane_sq(p, n, x) = T.0 * n.norm_sq.inverse
        T.0 * n.norm_sq.inverse = T.0
        point3_dist_to_plane_sq(p, n, x) = T.0
    }
}
