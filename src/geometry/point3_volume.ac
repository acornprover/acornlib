from comm_ring import CommRing
from geometry.point3 import Point3, point3_zero, point3_cross_self
from geometry.point3_algebra import point3_cross_norm_sq_identity,
    point3_scalar_triple_rotate, point3_scalar_triple_rotate2,
    point3_dot_zero_right

/// The squared area of the parallelogram spanned by two coordinate vectors.
define point3_parallelogram_area_sq[T: CommRing](u: Point3[T], v: Point3[T]) -> T {
    u.cross(v).norm_sq
}

/// The squared volume of the parallelepiped spanned by three coordinate vectors.
define point3_parallelepiped_volume_sq[T: CommRing](u: Point3[T], v: Point3[T], w: Point3[T]) -> T {
    u.dot(v.cross(w)) * u.dot(v.cross(w))
}

/// The squared parallelogram area is the Lagrange identity gap.
theorem point3_parallelogram_area_sq_eq_lagrange[T: CommRing](u: Point3[T], v: Point3[T]) {
    point3_parallelogram_area_sq(u, v) = u.norm_sq * v.norm_sq - u.dot(v) * u.dot(v)
} by {
    point3_cross_norm_sq_identity(u, v)
    u.cross(v).norm_sq = u.norm_sq * v.norm_sq - u.dot(v) * u.dot(v)
    point3_parallelogram_area_sq(u, v) = u.cross(v).norm_sq
}

/// The squared parallelepiped volume is unchanged by a cyclic rotation of the vectors.
theorem point3_parallelepiped_volume_sq_rotate[T: CommRing](u: Point3[T], v: Point3[T], w: Point3[T]) {
    point3_parallelepiped_volume_sq(u, v, w) = point3_parallelepiped_volume_sq(v, w, u)
} by {
    point3_scalar_triple_rotate(u, v, w)
    u.dot(v.cross(w)) = v.dot(w.cross(u))
    point3_parallelepiped_volume_sq(u, v, w) = u.dot(v.cross(w)) * u.dot(v.cross(w))
    point3_parallelepiped_volume_sq(v, w, u) = v.dot(w.cross(u)) * v.dot(w.cross(u))
}

/// The squared parallelepiped volume is unchanged by a double rotation of the vectors.
theorem point3_parallelepiped_volume_sq_rotate2[T: CommRing](u: Point3[T], v: Point3[T], w: Point3[T]) {
    point3_parallelepiped_volume_sq(u, v, w) = point3_parallelepiped_volume_sq(w, u, v)
} by {
    point3_scalar_triple_rotate2(u, v, w)
    u.dot(v.cross(w)) = w.dot(u.cross(v))
    point3_parallelepiped_volume_sq(u, v, w) = u.dot(v.cross(w)) * u.dot(v.cross(w))
    point3_parallelepiped_volume_sq(w, u, v) = w.dot(u.cross(v)) * w.dot(u.cross(v))
}

/// The squared volume of a degenerate parallelepiped with a repeated vector is zero.
theorem point3_parallelepiped_volume_sq_repeat[T: CommRing](u: Point3[T], v: Point3[T]) {
    point3_parallelepiped_volume_sq(u, v, v) = T.0
} by {
    point3_cross_self(v)
    v.cross(v) = point3_zero[T]
    point3_dot_zero_right(u)
    u.dot(point3_zero[T]) = T.0
    u.dot(v.cross(v)) = T.0
    point3_parallelepiped_volume_sq(u, v, v) = u.dot(v.cross(v)) * u.dot(v.cross(v))
    point3_parallelepiped_volume_sq(u, v, v) = T.0 * T.0
    T.0 * T.0 = T.0
}
