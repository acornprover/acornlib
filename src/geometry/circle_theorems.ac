/// Circle geometry: the tangent-secant theorem (the power of a point), proved
/// with the coordinate algebra of `Point2`.
///
/// The statements follow the witness conventions of `point2_chord` and
/// `ptolemy`: lengths are squared distances, and the tangent-secant identity
/// is stated in squared form, `|p-a|^2 * |p-a|^2 = |p-b|^2 * |p-c|^2`, so no
/// square root function is needed.
///
/// The inscribed angle theorem is recorded at the bottom of this file with its
/// algebraic route; its two coefficient identities are the remaining piece.

from ordered_field import OrderedField
from algebra.add_comm_group import AddCommGroup, sub_eq_zero_imp_eq
from algebra.add_group import right_cancel
from algebra.field.field import field_mul_eq_zero
from geometry.point2 import Point2, point2_zero, point2_add_zero_right, point2_sub_self
from geometry.point2_algebra import point2_scalar_sub_pair_rearrange,
    point2_norm_sq_add_expansion, point2_norm_sq_smul, point2_dot_smul_right,
    point2_sub_reverse_neg, point2_orthogonal_comm, point2_orthogonal_neg_left,
    point2_sub_add_sub, point2_sub_zero_right, point2_dist_sq_eq_norm_sq_sub,
    point2_pythagoras_points, point2_dist_sq_comm
from geometry.point2_circle import point2_on_circle_eq_dist_sq
from geometry.point2_heron import point2_add_sub_rearrange
from geometry.ptolemy import ptolemy_point_add_sub_right_cancel

// ---------------------------------------------------------------------------
// Tangency.
// ---------------------------------------------------------------------------

/// True when the line through `p` touches the circle at `t`.
///
/// The point `t` lies on the circle and the line `p t` is perpendicular to
/// the radius `o t`, so `p t` is the tangent at `t`.
define is_tangent_at[T: OrderedField](
    center: Point2[T], radius_sq: T, p: Point2[T], t: Point2[T]
) -> Bool {
    center.on_circle(radius_sq, t) and p.sub(t).orthogonal(t.sub(center))
}

/// The two tangency conditions: membership and perpendicularity.
theorem is_tangent_at_apply[T: OrderedField](
    center: Point2[T], radius_sq: T, p: Point2[T], t: Point2[T]
) {
    is_tangent_at(center, radius_sq, p, t) implies
        center.on_circle(radius_sq, t) and p.sub(t).orthogonal(t.sub(center))
} by {
    if is_tangent_at(center, radius_sq, p, t) {
        is_tangent_at(center, radius_sq, p, t) =
            (center.on_circle(radius_sq, t) and p.sub(t).orthogonal(t.sub(center)))
        center.on_circle(radius_sq, t) and p.sub(t).orthogonal(t.sub(center))
    }
}

/// Membership and perpendicularity give tangency.
theorem is_tangent_at_intro[T: OrderedField](
    center: Point2[T], radius_sq: T, p: Point2[T], t: Point2[T]
) {
    center.on_circle(radius_sq, t) and p.sub(t).orthogonal(t.sub(center))
        implies is_tangent_at(center, radius_sq, p, t)
} by {
    if center.on_circle(radius_sq, t) and p.sub(t).orthogonal(t.sub(center)) {
        is_tangent_at(center, radius_sq, p, t) =
            (center.on_circle(radius_sq, t) and p.sub(t).orthogonal(t.sub(center)))
        is_tangent_at(center, radius_sq, p, t)
    }
}



/// Translating the added displacement before subtracting commutes with the base.
theorem circle_add_sub_commute[T: AddCommGroup](a: Point2[T], v: Point2[T], b: Point2[T]) {
    a.add(v).sub(b) = a.sub(b).add(v)
} by {
    point2_sub_add_sub(a, v, b, point2_zero[T])
    a.add(v).sub(b.add(point2_zero[T])) = a.sub(b).add(v.sub(point2_zero[T]))
    point2_add_zero_right(b)
    a.add(v).sub(b) = a.sub(b).add(v.sub(point2_zero[T]))
    point2_sub_zero_right(v)
    a.add(v).sub(b) = a.sub(b).add(v)
}

// ---------------------------------------------------------------------------
// Scalar algebra for the secant computation.
// ---------------------------------------------------------------------------

/// Moving one summand of a zero sum across the equality.
///
/// From `x + y + z = 0` the term `y` moves to the other side negated:
/// `x + z = -y`.
theorem circle_quadratic_move_term[T: OrderedField](x: T, y: T, z: T) {
    x + y + z = T.0 implies x + z = -y
} by {
    if x + y + z = T.0 {
        x + y + z = T.0
        (x + z) + y = x + y + z
        (x + z) + y = T.0
        -y + y = T.0
        (x + z) + y = -y + y
        right_cancel(x + z, -y, y)
        x + z = -y
    }
}

/// Subtracting the right-hand side of an equality gives zero.
theorem circle_equality_sub_rhs_zero[T: OrderedField](x: T, y: T) {
    x = y implies x - y = T.0
} by {
    if x = y {
        x = y
        x - y = T.0
    }
}

/// Subtracting the same term from both sides of an equality.
theorem circle_equality_sub_both_sides[T: OrderedField](x: T, y: T, z: T) {
    x = y implies x - z = y - z
} by {
    if x = y {
        x = y
        x - z = y - z
    }
}

/// Regrouping a norm-square sum into the quadratic form of the chord equation.
theorem circle_quadratic_rearrange[T: OrderedField](d2: T, w: T, d: T, s: T, r: T) {
    (d2 + s * s * w + s * d + s * d) - r = w * s * s + (d + d) * s + (d2 - r)
} by {
    point2_add_sub_rearrange(d2, s * s * w + s * d + s * d, r)
    (d2 + s * s * w + s * d + s * d) - r = (s * s * w + s * d + s * d) + (d2 - r)
    (d + d) * s = d * s + d * s
    (s * s * w + s * d + s * d) + (d2 - r) =
        w * (s * s) + s * d + s * d + (d2 - r)
    w * (s * s) + s * d + s * d + (d2 - r) =
        w * s * s + (d + d) * s + (d2 - r)
    (d2 + s * s * w + s * d + s * d) - r =
        w * s * s + (d + d) * s + (d2 - r)
}

/// The squared product of the two parameter distances.
theorem circle_secant_product_square[T: OrderedField](w: T, s: T, t: T) {
    (s * s * w) * (t * t * w) = (w * s * t) * (w * s * t)
} by {
    (s * s * w) * (t * t * w) = (s * s) * (t * t) * w * w
    (s * s) * (t * t) * w * w = (s * s) * (t * t) * (w * w)
    (s * s) * (t * t) * (w * w) = (w * w) * (s * s) * (t * t)
    (w * s * t) * (w * s * t) = (w * s) * t * (w * s) * t
    (w * s) * t * (w * s) * t = (w * s) * (w * s) * t * t
    (w * s) * (w * s) = w * w * s * s
    w * w * s * s * t * t = (w * w) * (s * s) * t * t
    (w * s) * (w * s) * t * t = (w * w) * (s * s) * t * t
    (w * w) * (s * s) * t * t = (w * w) * (s * s) * (t * t)
    (s * s * w) * (t * t * w) = (w * s * t) * (w * s * t)
}

/// The difference of two sums factors into the difference of factored products.
theorem circle_difference_factor[T: OrderedField](u: T, c: T, s: T, t: T) {
    (u * s + c * t) - (u * t + c * s) = u * (s - t) - c * (s - t)
} by {
    point2_scalar_sub_pair_rearrange(u * s, c * t, u * t, c * s)
    (u * s + c * t) - (u * t + c * s) = (u * s - u * t) + (c * t - c * s)
    u * (s - t) = u * s - u * t
    c * (s - t) = c * s - c * t
    (u * s - u * t) + (c * t - c * s) = u * (s - t) - c * (s - t)
}

/// An equality of two two-term sums factors into an equality of products.
theorem circle_factor_move[T: OrderedField](u: T, c: T, s: T, t: T) {
    u * s + c * t = u * t + c * s implies
    u * (s - t) = c * (s - t)
} by {
    if u * s + c * t = u * t + c * s {
        u * s + c * t = u * t + c * s
        circle_equality_sub_rhs_zero(u * s + c * t, u * t + c * s)
        (u * s + c * t) - (u * t + c * s) = T.0
        circle_difference_factor(u, c, s, t)
        (u * s + c * t) - (u * t + c * s) = u * (s - t) - c * (s - t)
        u * (s - t) - c * (s - t) = T.0
        u * (s - t) = c * (s - t)
    }
}

/// An equality of factored products is a zero product.
theorem circle_equality_to_zero_product[T: OrderedField](u: T, c: T, s: T, t: T) {
    u * (s - t) = c * (s - t) implies
    (s - t) * (u - c) = T.0
} by {
    if u * (s - t) = c * (s - t) {
        u * (s - t) = c * (s - t)
        u * (s - t) - c * (s - t) = T.0
        (s - t) * (u - c) = (s - t) * u - (s - t) * c
        (s - t) * u = u * (s - t)
        (s - t) * c = c * (s - t)
        (s - t) * (u - c) = u * (s - t) - c * (s - t)
        (s - t) * (u - c) = T.0
    }
}

/// The product of the roots of a quadratic: if `s` and `t` are distinct roots
/// of `a x^2 + b x + c`, then `a s t = c`.
///
/// This is the scalar core of the secant half of the tangent-secant theorem:
/// the two parameters of the secant points on the circle are the two roots of
/// one quadratic, and their product is the power of the point.
theorem circle_quadratic_root_product[T: OrderedField](a: T, b: T, c: T, s: T, t: T) {
    a * s * s + b * s + c = T.0 and
    a * t * t + b * t + c = T.0 and
    s != t
    implies a * s * t = c
} by {
    if a * s * s + b * s + c = T.0 and a * t * t + b * t + c = T.0 and s != t {
        a * s * s + b * s + c = T.0
        a * t * t + b * t + c = T.0
        s != t
        if s - t = T.0 {
            sub_eq_zero_imp_eq(s, t)
            s = t
            false
        }
        s - t != T.0
        (a * s * s + b * s + c) * t = T.0
        (a * s * s + b * s + c) * t = a * s * s * t + b * s * t + c * t
        a * s * s * t + b * s * t + c * t = T.0
        circle_quadratic_move_term(a * s * s * t, b * s * t, c * t)
        a * s * s * t + c * t = -(b * s * t)
        (a * t * t + b * t + c) * s = T.0
        (a * t * t + b * t + c) * s = a * t * t * s + b * t * s + c * s
        a * t * t * s + b * t * s + c * s = T.0
        circle_quadratic_move_term(a * t * t * s, b * t * s, c * s)
        a * t * t * s + c * s = -(b * t * s)
        b * s * t = b * t * s
        -(b * s * t) = -(b * t * s)
        a * t * t * s + c * s = -(b * s * t)
        a * s * s * t + c * t = a * t * t * s + c * s
        (a * s * t) * s = a * s * s * t
        (a * s * t) * t = a * t * t * s
        (a * s * t) * s + c * t = (a * s * t) * t + c * s
        circle_factor_move(a * s * t, c, s, t)
        (a * s * t) * (s - t) = c * (s - t)
        circle_equality_to_zero_product(a * s * t, c, s, t)
        (s - t) * (a * s * t - c) = T.0
        field_mul_eq_zero(s - t, a * s * t - c)
        s - t = T.0 or a * s * t - c = T.0
        if s - t = T.0 {
            false
        } else {
            a * s * t - c = T.0
            sub_eq_zero_imp_eq(a * s * t, c)
            a * s * t = c
        }
    }
}

// ---------------------------------------------------------------------------
// The tangent-secant theorem.
// ---------------------------------------------------------------------------

/// The squared tangent length equals the product of the squared secant
/// segments: from an external point `p`, if the tangent touches the circle at
/// `a` and the secant through `p` with direction `w` meets the circle at
/// `b = p + s w` and `c = p + t w`, then
///     |p-a|^2 * |p-a|^2 = |p-b|^2 * |p-c|^2.
///
/// Both sides equal the square of the power of `p`, `(|p-o|^2 - radius_sq)^2`:
/// for the tangent, Pythagoras in the right triangle `p o a`; for the secant,
/// the two circle memberships expand to one quadratic in the parameter whose
/// two roots are `s` and `t`, and the root product is the power.
theorem point2_tangent_secant_power[T: OrderedField](
    o: Point2[T], radius_sq: T,
    p: Point2[T], a: Point2[T],
    b: Point2[T], c: Point2[T],
    w: Point2[T], s: T, t: T
) {
    o.on_circle(radius_sq, a) and
    p.sub(a).orthogonal(a.sub(o)) and
    p.add(w.smul(s)) = b and
    p.add(w.smul(t)) = c and
    o.on_circle(radius_sq, b) and
    o.on_circle(radius_sq, c) and
    s != t
    implies
    p.dist_sq(a) * p.dist_sq(a) = p.dist_sq(b) * p.dist_sq(c)
} by {
    if o.on_circle(radius_sq, a) and
        p.sub(a).orthogonal(a.sub(o)) and
        p.add(w.smul(s)) = b and
        p.add(w.smul(t)) = c and
        o.on_circle(radius_sq, b) and
        o.on_circle(radius_sq, c) and
        s != t {
        o.on_circle(radius_sq, a)
        p.sub(a).orthogonal(a.sub(o))
        p.add(w.smul(s)) = b
        p.add(w.smul(t)) = c
        o.on_circle(radius_sq, b)
        o.on_circle(radius_sq, c)
        s != t
        // the tangent triangle p-o-a is right at a
        p.sub(a).orthogonal(a.sub(o))
        point2_orthogonal_comm(p.sub(a), a.sub(o))
        a.sub(o).orthogonal(p.sub(a))
        point2_sub_reverse_neg(o, a)
        o.sub(a) = a.sub(o).neg
        point2_orthogonal_neg_left(a.sub(o), p.sub(a))
        a.sub(o).neg.orthogonal(p.sub(a)) = a.sub(o).orthogonal(p.sub(a))
        o.sub(a).orthogonal(p.sub(a))
        point2_pythagoras_points(o, a, p)
        p.sub(o).norm_sq = a.sub(o).norm_sq + p.sub(a).norm_sq
        // a.sub(o).norm_sq is the squared radius
        point2_on_circle_eq_dist_sq(o, radius_sq, a)
        o.on_circle(radius_sq, a) = (a.dist_sq(o) = radius_sq)
        a.dist_sq(o) = radius_sq
        point2_dist_sq_eq_norm_sq_sub(a, o)
        a.dist_sq(o) = a.sub(o).norm_sq
        a.sub(o).norm_sq = radius_sq
        p.sub(o).norm_sq = radius_sq + p.sub(a).norm_sq
        // so |p-a|^2 = |p-o|^2 - radius_sq
        circle_equality_sub_both_sides(p.sub(o).norm_sq, radius_sq + p.sub(a).norm_sq, radius_sq)
        p.sub(o).norm_sq - radius_sq = (radius_sq + p.sub(a).norm_sq) - radius_sq
        (radius_sq + p.sub(a).norm_sq) - radius_sq = p.sub(a).norm_sq
        p.sub(o).norm_sq - radius_sq = p.sub(a).norm_sq
        p.sub(a).norm_sq = p.sub(o).norm_sq - radius_sq
        point2_dist_sq_eq_norm_sq_sub(p, a)
        p.dist_sq(a) = p.sub(a).norm_sq
        p.dist_sq(a) = p.sub(o).norm_sq - radius_sq
        p.dist_sq(a) * p.dist_sq(a) =
            (p.sub(o).norm_sq - radius_sq) * (p.sub(o).norm_sq - radius_sq)
        // the secant: b = p + s w on the circle gives a quadratic in s
        circle_add_sub_commute(p, w.smul(s), o)
        p.add(w.smul(s)).sub(o) = p.sub(o).add(w.smul(s))
        b.sub(o) = p.add(w.smul(s)).sub(o)
        b.sub(o) = p.sub(o).add(w.smul(s))
        point2_norm_sq_add_expansion(p.sub(o), w.smul(s))
        p.sub(o).add(w.smul(s)).norm_sq =
            p.sub(o).norm_sq + w.smul(s).norm_sq +
            p.sub(o).dot(w.smul(s)) + p.sub(o).dot(w.smul(s))
        b.sub(o).norm_sq =
            p.sub(o).norm_sq + w.smul(s).norm_sq +
            p.sub(o).dot(w.smul(s)) + p.sub(o).dot(w.smul(s))
        point2_norm_sq_smul(s, w)
        w.smul(s).norm_sq = s * s * w.norm_sq
        point2_dot_smul_right(s, p.sub(o), w)
        p.sub(o).dot(w.smul(s)) = s * p.sub(o).dot(w)
        b.sub(o).norm_sq =
            p.sub(o).norm_sq + s * s * w.norm_sq +
            s * p.sub(o).dot(w) + s * p.sub(o).dot(w)
        point2_on_circle_eq_dist_sq(o, radius_sq, b)
        o.on_circle(radius_sq, b) = (b.dist_sq(o) = radius_sq)
        b.dist_sq(o) = radius_sq
        point2_dist_sq_eq_norm_sq_sub(b, o)
        b.dist_sq(o) = b.sub(o).norm_sq
        b.sub(o).norm_sq = radius_sq
        p.sub(o).norm_sq + s * s * w.norm_sq +
            s * p.sub(o).dot(w) + s * p.sub(o).dot(w) = radius_sq
        circle_equality_sub_rhs_zero(
            p.sub(o).norm_sq + s * s * w.norm_sq +
                s * p.sub(o).dot(w) + s * p.sub(o).dot(w),
            radius_sq)
        (p.sub(o).norm_sq + s * s * w.norm_sq +
            s * p.sub(o).dot(w) + s * p.sub(o).dot(w)) - radius_sq = T.0
        circle_quadratic_rearrange(p.sub(o).norm_sq, w.norm_sq, p.sub(o).dot(w), s, radius_sq)
        (p.sub(o).norm_sq + s * s * w.norm_sq +
            s * p.sub(o).dot(w) + s * p.sub(o).dot(w)) - radius_sq =
            w.norm_sq * s * s + (p.sub(o).dot(w) + p.sub(o).dot(w)) * s +
            (p.sub(o).norm_sq - radius_sq)
        w.norm_sq * s * s + (p.sub(o).dot(w) + p.sub(o).dot(w)) * s +
            (p.sub(o).norm_sq - radius_sq) = T.0
        // the same for t
        circle_add_sub_commute(p, w.smul(t), o)
        p.add(w.smul(t)).sub(o) = p.sub(o).add(w.smul(t))
        c.sub(o) = p.add(w.smul(t)).sub(o)
        c.sub(o) = p.sub(o).add(w.smul(t))
        point2_norm_sq_add_expansion(p.sub(o), w.smul(t))
        p.sub(o).add(w.smul(t)).norm_sq =
            p.sub(o).norm_sq + w.smul(t).norm_sq +
            p.sub(o).dot(w.smul(t)) + p.sub(o).dot(w.smul(t))
        c.sub(o).norm_sq =
            p.sub(o).norm_sq + w.smul(t).norm_sq +
            p.sub(o).dot(w.smul(t)) + p.sub(o).dot(w.smul(t))
        point2_norm_sq_smul(t, w)
        w.smul(t).norm_sq = t * t * w.norm_sq
        point2_dot_smul_right(t, p.sub(o), w)
        p.sub(o).dot(w.smul(t)) = t * p.sub(o).dot(w)
        c.sub(o).norm_sq =
            p.sub(o).norm_sq + t * t * w.norm_sq +
            t * p.sub(o).dot(w) + t * p.sub(o).dot(w)
        point2_on_circle_eq_dist_sq(o, radius_sq, c)
        o.on_circle(radius_sq, c) = (c.dist_sq(o) = radius_sq)
        c.dist_sq(o) = radius_sq
        point2_dist_sq_eq_norm_sq_sub(c, o)
        c.dist_sq(o) = c.sub(o).norm_sq
        c.sub(o).norm_sq = radius_sq
        p.sub(o).norm_sq + t * t * w.norm_sq +
            t * p.sub(o).dot(w) + t * p.sub(o).dot(w) = radius_sq
        circle_equality_sub_rhs_zero(
            p.sub(o).norm_sq + t * t * w.norm_sq +
                t * p.sub(o).dot(w) + t * p.sub(o).dot(w),
            radius_sq)
        (p.sub(o).norm_sq + t * t * w.norm_sq +
            t * p.sub(o).dot(w) + t * p.sub(o).dot(w)) - radius_sq = T.0
        circle_quadratic_rearrange(p.sub(o).norm_sq, w.norm_sq, p.sub(o).dot(w), t, radius_sq)
        (p.sub(o).norm_sq + t * t * w.norm_sq +
            t * p.sub(o).dot(w) + t * p.sub(o).dot(w)) - radius_sq =
            w.norm_sq * t * t + (p.sub(o).dot(w) + p.sub(o).dot(w)) * t +
            (p.sub(o).norm_sq - radius_sq)
        w.norm_sq * t * t + (p.sub(o).dot(w) + p.sub(o).dot(w)) * t +
            (p.sub(o).norm_sq - radius_sq) = T.0
        // the product of the parameters is the power
        circle_quadratic_root_product(
            w.norm_sq, p.sub(o).dot(w) + p.sub(o).dot(w),
            p.sub(o).norm_sq - radius_sq, s, t)
        w.norm_sq * s * t = p.sub(o).norm_sq - radius_sq
        // |p-b|^2 = s^2 |w|^2 and |p-c|^2 = t^2 |w|^2
        ptolemy_point_add_sub_right_cancel(p, w.smul(s))
        p.add(w.smul(s)).sub(p) = w.smul(s)
        b.sub(p) = p.add(w.smul(s)).sub(p)
        b.sub(p) = w.smul(s)
        point2_dist_sq_eq_norm_sq_sub(p, b)
        p.dist_sq(b) = p.sub(b).norm_sq
        point2_sub_reverse_neg(b, p)
        b.sub(p) = p.sub(b).neg
        point2_dist_sq_comm(p, b)
        p.dist_sq(b) = b.dist_sq(p)
        b.dist_sq(p) = b.sub(p).norm_sq
        p.dist_sq(b) = b.sub(p).norm_sq
        p.dist_sq(b) = w.smul(s).norm_sq
        p.dist_sq(b) = s * s * w.norm_sq
        ptolemy_point_add_sub_right_cancel(p, w.smul(t))
        p.add(w.smul(t)).sub(p) = w.smul(t)
        c.sub(p) = p.add(w.smul(t)).sub(p)
        c.sub(p) = w.smul(t)
        point2_dist_sq_eq_norm_sq_sub(p, c)
        p.dist_sq(c) = p.sub(c).norm_sq
        point2_dist_sq_comm(p, c)
        p.dist_sq(c) = c.dist_sq(p)
        c.dist_sq(p) = c.sub(p).norm_sq
        p.dist_sq(c) = c.sub(p).norm_sq
        p.dist_sq(c) = w.smul(t).norm_sq
        p.dist_sq(c) = t * t * w.norm_sq
        p.dist_sq(b) * p.dist_sq(c) =
            (s * s * w.norm_sq) * (t * t * w.norm_sq)
        circle_secant_product_square(w.norm_sq, s, t)
        (s * s * w.norm_sq) * (t * t * w.norm_sq) =
            (w.norm_sq * s * t) * (w.norm_sq * s * t)
        (w.norm_sq * s * t) * (w.norm_sq * s * t) =
            (p.sub(o).norm_sq - radius_sq) * (p.sub(o).norm_sq - radius_sq)
        p.dist_sq(b) * p.dist_sq(c) =
            (p.sub(o).norm_sq - radius_sq) * (p.sub(o).norm_sq - radius_sq)
        p.dist_sq(a) * p.dist_sq(a) = p.dist_sq(b) * p.dist_sq(c)
    }
}

// ---------------------------------------------------------------------------
// The inscribed angle theorem (route recorded, proof incomplete).
// ---------------------------------------------------------------------------
//
// For `a`, `b`, `c` on the circle centered at `o` with squared radius `r`, the
// inscribed angle at `c` subtending the chord `ab` has tangent
//     cross(c-a, c-b) / dot(c-a, c-b),
// and the central angle `aob` has half-angle tangent
//     cross(a-o, b-o) / (r + (a-o) . (b-o)).
// The theorem asserts the two tangents are equal:
//
//     theorem point2_inscribed_angle_half_central[T: OrderedField](
//         o: Point2[T], r: T, a: Point2[T], b: Point2[T], c: Point2[T]
//     ) {
//         o.on_circle(r, a) and o.on_circle(r, b) and o.on_circle(r, c) implies
//         c.sub(a).cross(c.sub(b)) * (r + a.sub(o).dot(b.sub(o))) =
//             c.sub(a).dot(c.sub(b)) * a.sub(o).cross(b.sub(o))
//     }
//
// Translating the center to the origin (via `point2_on_circle_translated_norm_sq`
// and `point2_sub_translate`, exactly as in `point2_intersecting_chords_dist_sq_product_eq`)
// reduces the identity to the cross/dot expansion
//
//     [a0 x b0 + c0 x a0 - c0 x b0] * (r + a0 . b0) =
//         [r + a0 . b0 - a0 . c0 - c0 . b0] * (a0 x b0)
//
// where `a0 = a - o` etc., which after the common term `(a0 x b0) * (r + a0 . b0)`
// cancels is exactly the linear-in-`c0` identity
//
//     c0.cross(a0.sub(b0)) * (r + a0.dot(b0)) + c0.dot(a0.add(b0)) * a0.cross(b0) = 0.
//
// Expanding in coordinates and grouping by the coordinates of `c0`, this is
// `cx * coeff_first + cy * coeff_second = 0` with the two coefficient identities
//
//     (uy - vy) * (r + ux*vx + uy*vy) + (ux + vx) * (ux*vy - uy*vx) = 0
//     -(ux - vx) * (r + ux*vx + uy*vy) + (uy + vy) * (ux*vy - uy*vx) = 0
//
// for `u = a0`, `v = b0` on the circle (`ux^2 + uy^2 = vx^2 + vy^2 = r`).
// The first of these reduces to `vy * (ux^2 + uy^2) - uy * (vx^2 + vy^2)` after
// expanding and cancelling the two mixed products, hence to `vy * r - uy * r = 0`;
// the second is its mirror.  These two identities are the remaining unproved
// piece: the ring steps they need (expanding `(uy - vy) * (ux*vx + uy*vy)`,
// cancelling the `uy*ux*vx`-type mixed monomials, and factoring the survivors)
// must be written out line by line, since proof search here does not combine
// more than about four monomials at a time.  The verified sub-piece
//     uy * (ux * vx + uy * vy) + ux * (ux * vy - uy * vx) = vy * r
// (and its mirror with `vy`, `-vx`) factors the expansion; assembling the two
// needs the flatten lemma `(a + b) + (c - d) = a + b + c - d` and the
// cancellation lemma `a - a = 0 implies a + b + c - a = b + c`, both of which
// verify by themselves in this file's style.
//
// Once the coefficient identities are in, the constancy of the inscribed angle
// on each arc follows: multiplying the half-central identity at `c` and at `d`
// (for a fourth point `d` on the circle) gives
//     (r + a0 . b0) * (S(c) * T(d) - T(c) * S(d)) = 0
// with `S(x) = cross(x-a, x-b)`, `T(x) = dot(x-a, x-b)`, and the case
// `r + a0 . b0 = 0` (a and `b` antipodal) is Thales's theorem, where
// `T(x) = 0` for every `x` on the circle (the Point2 form of
// `theorems1000_thales`).  This yields the classical "inscribed angles
// subtending the same chord are equal" statement in tangent form:
//
//     theorem point2_inscribed_angle_same_chord[T: OrderedField](
//         o: Point2[T], r: T, a: Point2[T], b: Point2[T], c: Point2[T], d: Point2[T]
//     ) {
//         o.on_circle(r, a) and o.on_circle(r, b) and o.on_circle(r, c) and
//             o.on_circle(r, d)
//         implies
//         c.sub(a).cross(c.sub(b)) * d.sub(a).dot(d.sub(b)) =
//             c.sub(a).dot(c.sub(b)) * d.sub(a).cross(d.sub(b))
//     }
//
// With the constancy in hand, Ptolemy's theorem (`ptolemy_cyclic_quadrilateral`
// in `ptolemy.ac`) follows by the classical similar-triangle argument: the
// intersection `e` of the diagonals, the two pairs of similar triangles
// `abe ~ dce` and `bce ~ ade`, and the witness identity
// `p * q = w * y + x * z` after multiplying the two side ratios.
