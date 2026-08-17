/// Triangle geometry: the triangle inequality, the Pythagorean theorem, the
/// base-times-height area law, and Heron's formula, with the 3-4-5 right
/// triangle as the running example.
///
/// Status:
///   - The triangle inequality is proved for one-dimensional real points — the
///     absolute-value inequality |x - z| ≤ |x - y| + |y - z|.  The full planar
///     inequality for three points (Freek Top 100 #91) needs the
///     Cauchy-Schwarz machinery for the coordinate plane and is recorded in a
///     comment below, not re-proved here.
///   - The Pythagorean theorem is proved for right triangles in the coordinate
///     plane in squared-distance form, and the 3-4-5 instance 3² + 4² = 5² is
///     verified.
///   - The (1/2)·base·height area law is stated, the squared doubled area of a
///     right triangle is tied to the product of the legs, and the area of the
///     3-4-5 right triangle is verified to be 6.
///   - Heron's formula is proved in its squared (doubled-area) form, and the
///     3-4-5 instance — s = 6 and √(s(s-a)(s-b)(s-c)) = √(6·3·2·1) = √36 = 6 —
///     is verified.

from real import Real, triangle_ineq, sqrt_mul_self, sqrt_value_nonneg, square_le_square_of_nonneg,
    product_pair_square_rearrange
from order import lte_antisymm
from geometry.point2 import Point2
from geometry.point2_algebra import point2_pythagoras_points, point2_scalar_sub_through
from geometry.point2_metric import point2_dist_sq_eq_norm_sq_sub_reverse,
    point2_dist_sq_comm
from geometry.point2_heron import point2_heron_triangle_squared, heron_linear_product,
    point2_lagrange_identity
from geometry.point2_triangle import point2_triangle_area2_eq_orientation
from geometry.triangle_geometry_arith import real_two, real_three, real_four, real_five,
    real_six, real_twelve, real_thirty_six, real_six_pos, real_six_nonneg,
    real_two_ne_zero, real_three_add_four_add_five_is_twelve,
    real_six_sub_three_is_three, real_six_sub_four_is_two, real_six_sub_five_is_one,
    real_two_mul_two_is_four, real_three_mul_two_is_six, real_four_mul_two_is_eight,
    real_six_mul_two_is_twelve, real_three_mul_four_is_twelve,
    real_six_mul_three_is_eighteen, real_eighteen_mul_two_is_thirty_six,
    real_six_mul_six_is_thirty_six, real_three_sq_is_nine, real_four_sq_is_sixteen,
    real_five_sq_is_twentyfive, real_nine_add_sixteen_is_twentyfive,
    real_six_mul_three_mul_two_mul_one_is_thirty_six, real_twelve_div_two_is_six,
    real_two_add_three_is_five, real_three_add_three_is_six,
    real_four_add_five_is_nine, real_nine_add_three_is_twelve

numerals Real

// ---------------------------------------------------------------------------
// The triangle and its sides.
// ---------------------------------------------------------------------------

/// True when `x`, `y`, `z` are the nonnegative lengths of the three sides of
/// the triangle with vertices `a`, `b`, `c` — `x` opposite `a` (the side
/// `bc`), `y` opposite `b` (the side `ca`), and `z` opposite `c` (the side
/// `ab`) — witnessed through the squared distances between the vertices.
define is_triangle(
    a: Point2[Real], b: Point2[Real], c: Point2[Real], x: Real, y: Real, z: Real
) -> Bool {
    Real.0 <= x and Real.0 <= y and Real.0 <= z and
    x * x = b.dist_sq(c) and y * y = c.dist_sq(a) and z * z = a.dist_sq(b)
}

/// The defining equation of `is_triangle`.
theorem is_triangle_eq(
    a: Point2[Real], b: Point2[Real], c: Point2[Real], x: Real, y: Real, z: Real
) {
    is_triangle(a, b, c, x, y, z) =
        (Real.0 <= x and Real.0 <= y and Real.0 <= z and
         x * x = b.dist_sq(c) and y * y = c.dist_sq(a) and z * z = a.dist_sq(b))
} by { }

/// The side-length squares of a triangle are the squared norms of the side
/// displacements, in the order used by the Heron identities.
theorem triangle_sides_as_norm_sq(
    a: Point2[Real], b: Point2[Real], c: Point2[Real], x: Real, y: Real, z: Real
) {
    is_triangle(a, b, c, x, y, z) implies
        (x * x = c.sub(b).norm_sq and y * y = c.sub(a).norm_sq and z * z = b.sub(a).norm_sq)
} by {
    if is_triangle(a, b, c, x, y, z) {
        is_triangle_eq(a, b, c, x, y, z)
        is_triangle(a, b, c, x, y, z) =
            (Real.0 <= x and Real.0 <= y and Real.0 <= z and
             x * x = b.dist_sq(c) and y * y = c.dist_sq(a) and z * z = a.dist_sq(b))
        x * x = b.dist_sq(c) and y * y = c.dist_sq(a) and z * z = a.dist_sq(b)
        x * x = b.dist_sq(c)
        y * y = c.dist_sq(a)
        z * z = a.dist_sq(b)
        point2_dist_sq_eq_norm_sq_sub_reverse(b, c)
        b.dist_sq(c) = c.sub(b).norm_sq
        x * x = c.sub(b).norm_sq
        point2_dist_sq_comm(c, a)
        c.dist_sq(a) = a.dist_sq(c)
        point2_dist_sq_eq_norm_sq_sub_reverse(a, c)
        a.dist_sq(c) = c.sub(a).norm_sq
        y * y = c.sub(a).norm_sq
        point2_dist_sq_eq_norm_sq_sub_reverse(a, b)
        a.dist_sq(b) = b.sub(a).norm_sq
        z * z = b.sub(a).norm_sq
        x * x = c.sub(b).norm_sq and y * y = c.sub(a).norm_sq and z * z = b.sub(a).norm_sq
    }
}

// ---------------------------------------------------------------------------
// The triangle inequality.
// ---------------------------------------------------------------------------

/// The 1-dimensional triangle inequality for the real absolute value:
/// the distance between `x` and `z` is at most the sum of the distances
/// through `y`.
theorem real_triangle_inequality(x: Real, y: Real, z: Real) {
    (x - z).abs <= (x - y).abs + (y - z).abs
} by {
    point2_scalar_sub_through[Real](z, y, x)
    (y - z) + (x - y) = x - z
    (y - z) + (x - y) = (x - y) + (y - z)
    (x - y) + (y - z) = x - z
    triangle_ineq(x - y, y - z)
    ((x - y) + (y - z)).abs <= (x - y).abs + (y - z).abs
    (x - z).abs <= (x - y).abs + (y - z).abs
}

// The planar triangle inequality — for any three points of the coordinate
// plane the distance between two of them is at most the sum of the other two
// distances — is Freek Top 100 #91.  With the length witnesses of
// `is_triangle` it reads
//
//     is_triangle(a, b, c, x, y, z) implies x <= y + z
//
// Its proof needs the Cauchy-Schwarz inequality for the two coordinate
// sequences (`finite_cauchy_schwarz` in src/real/cauchy_schwarz.ac) exactly
// as in src/top100/theorem_091_triangle_inequality.ac, where the planar
// inequality is proved in norm form.  It is not re-proved here.

// ---------------------------------------------------------------------------
// The Pythagorean theorem.
// ---------------------------------------------------------------------------

/// The Pythagorean theorem: in a right triangle with legs `x` and `z` and
/// hypotenuse `y` (the right angle at `b`), the square of the hypotenuse is
/// the sum of the squares of the legs.
theorem pythagoras_of_right_triangle(
    a: Point2[Real], b: Point2[Real], c: Point2[Real], x: Real, y: Real, z: Real
) {
    is_triangle(a, b, c, x, y, z) and a.sub(b).orthogonal(c.sub(b)) implies
    x * x + z * z = y * y
} by {
    if is_triangle(a, b, c, x, y, z) and a.sub(b).orthogonal(c.sub(b)) {
        triangle_sides_as_norm_sq(a, b, c, x, y, z)
        x * x = c.sub(b).norm_sq and y * y = c.sub(a).norm_sq and z * z = b.sub(a).norm_sq
        x * x = c.sub(b).norm_sq
        y * y = c.sub(a).norm_sq
        z * z = b.sub(a).norm_sq
        point2_pythagoras_points(a, b, c)
        c.sub(a).norm_sq = b.sub(a).norm_sq + c.sub(b).norm_sq
        y * y = b.sub(a).norm_sq + c.sub(b).norm_sq
        b.sub(a).norm_sq + c.sub(b).norm_sq = z * z + x * x
        y * y = z * z + x * x
        x * x + z * z = y * y
    }
}

/// Three squared plus four squared equals five squared: the 3-4-5 instance of
/// the Pythagorean theorem.
theorem pythagoras_three_four_five {
    real_three * real_three + real_four * real_four = real_five * real_five
} by { }

// ---------------------------------------------------------------------------
// The area of a triangle.
// ---------------------------------------------------------------------------

/// The area of a triangle with base `base` and height `height`: one half of
/// the product of the base and the height.
define triangle_area(base: Real, height: Real) -> Real {
    base * height / real_two
}

/// The area of a triangle with base `base` and height `height` is half the
/// product of the two.
theorem triangle_area_half_base_height(base: Real, height: Real) {
    triangle_area(base, height) = base * height / real_two
} by {
    triangle_area(base, height) = base * height / real_two
}

/// The squared doubled area of a right triangle is the square of the product
/// of its legs: if the legs of the right angle at `a` have squared lengths
/// `base²` and `height²`, then the signed doubled area has square
/// `(base·height)²`.
theorem right_triangle_doubled_area_squared(
    a: Point2[Real], b: Point2[Real], c: Point2[Real], base: Real, height: Real
) {
    a.dist_sq(b) = base * base and a.dist_sq(c) = height * height and
        b.sub(a).orthogonal(c.sub(a))
    implies
    (a.triangle_area2(b, c)) * (a.triangle_area2(b, c)) =
        (base * height) * (base * height)
} by {
    if a.dist_sq(b) = base * base and a.dist_sq(c) = height * height and
        b.sub(a).orthogonal(c.sub(a)) {
        point2_triangle_area2_eq_orientation(a, b, c)
        a.triangle_area2(b, c) = b.sub(a).cross(c.sub(a))
        point2_lagrange_identity(b.sub(a), c.sub(a))
        b.sub(a).cross(c.sub(a)) * b.sub(a).cross(c.sub(a)) +
            b.sub(a).dot(c.sub(a)) * b.sub(a).dot(c.sub(a)) =
            b.sub(a).norm_sq * c.sub(a).norm_sq
        b.sub(a).dot(c.sub(a)) = Real.0
        b.sub(a).dot(c.sub(a)) * b.sub(a).dot(c.sub(a)) = Real.0
        b.sub(a).cross(c.sub(a)) * b.sub(a).cross(c.sub(a)) + Real.0 =
            b.sub(a).norm_sq * c.sub(a).norm_sq
        b.sub(a).cross(c.sub(a)) * b.sub(a).cross(c.sub(a)) =
            b.sub(a).norm_sq * c.sub(a).norm_sq
        point2_dist_sq_eq_norm_sq_sub_reverse(a, b)
        a.dist_sq(b) = b.sub(a).norm_sq
        b.sub(a).norm_sq = a.dist_sq(b)
        point2_dist_sq_eq_norm_sq_sub_reverse(a, c)
        a.dist_sq(c) = c.sub(a).norm_sq
        c.sub(a).norm_sq = a.dist_sq(c)
        b.sub(a).norm_sq * c.sub(a).norm_sq = a.dist_sq(b) * a.dist_sq(c)
        a.dist_sq(b) * a.dist_sq(c) = base * base * (height * height)
        product_pair_square_rearrange(base, height)
        (base * height) * (base * height) = (base * base) * (height * height)
        b.sub(a).norm_sq * c.sub(a).norm_sq = (base * height) * (base * height)
        a.triangle_area2(b, c) * a.triangle_area2(b, c) =
            (base * height) * (base * height)
    }
}

/// The area of the 3-4-5 right triangle is six.
theorem right_triangle_area_is_six {
    triangle_area(real_three, real_four) = real_six
} by { }

// ---------------------------------------------------------------------------
// Heron's formula.
// ---------------------------------------------------------------------------

/// The semiperimeter of a triangle with side lengths `x`, `y`, `z`.
define semiperimeter(x: Real, y: Real, z: Real) -> Real {
    (x + y + z) / real_two
}

/// The Heron radicand: the product of the semiperimeter and the three
/// semiperimeter-minus-side factors, whose square root Heron's formula takes.
define heron_radicand(x: Real, y: Real, z: Real) -> Real {
    semiperimeter(x, y, z) * (semiperimeter(x, y, z) - x) *
    (semiperimeter(x, y, z) - y) * (semiperimeter(x, y, z) - z)
}

/// The area of a triangle with side lengths `x`, `y`, `z` by Heron's formula:
/// the square root of the Heron radicand.
define heron_area(x: Real, y: Real, z: Real) -> Option[Real] {
    (heron_radicand(x, y, z)).sqrt
}

/// Heron's formula in squared doubled-area form: for a triangle with side
/// lengths `x`, `y`, `z`, the square of twice the (signed) area equals the
/// product of the four Heron factors (x+y+z)(-x+y+z)(x-y+z)(x+y-z).
///
/// With `s = (x + y + z) / 2` the four factors are `2s`, `2(s-x)`, `2(s-y)`,
/// `2(s-z)`, so the identity reads `(2A)² = 16·s(s-x)(s-y)(s-z)`, i.e.
/// `(A/2)² = s(s-x)(s-y)(s-z)` for the ordinary area `A/2`; taking the square
/// root gives the classical `area = √(s(s-x)(s-y)(s-z))`.
theorem heron_formula_squared(
    a: Point2[Real], b: Point2[Real], c: Point2[Real], x: Real, y: Real, z: Real
) {
    is_triangle(a, b, c, x, y, z) implies
    heron_linear_product(x, y, z) =
        (a.triangle_area2(b, c) + a.triangle_area2(b, c)) *
        (a.triangle_area2(b, c) + a.triangle_area2(b, c))
} by {
    if is_triangle(a, b, c, x, y, z) {
        triangle_sides_as_norm_sq(a, b, c, x, y, z)
        x * x = c.sub(b).norm_sq and y * y = c.sub(a).norm_sq and z * z = b.sub(a).norm_sq
        point2_heron_triangle_squared(a, b, c, x, y, z)
        heron_linear_product(x, y, z) =
            (a.triangle_area2(b, c) + a.triangle_area2(b, c)) *
            (a.triangle_area2(b, c) + a.triangle_area2(b, c))
    }
}

/// The semiperimeter of the 3-4-5 triangle is six.
theorem semiperimeter_345_is_six {
    semiperimeter(real_three, real_four, real_five) = real_six
} by { }

/// The Heron radicand of the 3-4-5 triangle is thirty-six.
theorem heron_radicand_345_is_thirty_six {
    heron_radicand(real_three, real_four, real_five) = real_thirty_six
} by { }

/// The square root of thirty-six is six.
theorem sqrt_thirty_six_is_six {
    real_thirty_six.sqrt = Option.some(real_six)
} by {
    real_six_mul_six_is_thirty_six
    real_six * real_six = real_thirty_six
    real_thirty_six = real_six * real_six
    real_six_nonneg
    real_six >= Real.0
    sqrt_mul_self(real_six * real_six)
    exists(y: Real) {
        (real_six * real_six).sqrt = Option.some(y) and y * y = real_six * real_six
    }
    let y: Real satisfy {
        (real_six * real_six).sqrt = Option.some(y) and y * y = real_six * real_six
    }
    sqrt_value_nonneg(real_six * real_six, y)
    y >= Real.0
    y * y = real_six * real_six
    y * y = real_thirty_six
    square_le_square_of_nonneg(y, real_six)
    y <= real_six
    square_le_square_of_nonneg(real_six, y)
    real_six <= y
    lte_antisymm(real_six, y)
    real_six = y
    y = real_six
    (real_six * real_six).sqrt = Option.some(real_six)
    real_thirty_six.sqrt = Option.some(real_six)
}

/// For the 3-4-5 right triangle, Heron's formula gives area six:
/// s = (3 + 4 + 5) / 2 = 6 and √(s(s-a)(s-b)(s-c)) = √(6·3·2·1) = √36 = 6.
theorem heron_three_four_five {
    heron_area(real_three, real_four, real_five) = Option.some(real_six)
} by {
    heron_area(real_three, real_four, real_five) =
        (heron_radicand(real_three, real_four, real_five)).sqrt
    heron_radicand_345_is_thirty_six
    heron_radicand(real_three, real_four, real_five) = real_thirty_six
    (heron_radicand(real_three, real_four, real_five)).sqrt = real_thirty_six.sqrt
    sqrt_thirty_six_is_six
    real_thirty_six.sqrt = Option.some(real_six)
    heron_area(real_three, real_four, real_five) = Option.some(real_six)
}
