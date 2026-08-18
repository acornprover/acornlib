from nat import Nat, from_nat, div_mul, zero_div
from real import Real
from pair import Pair, pair_ext, pair_new_first, pair_new_second
from geometry.point2 import Point2, point2_ext, point2_new_x, point2_new_y
from geometry.point2_algebra import point2_dist_sq_self, point2_dist_sq_comm
from geometry.point2_incidence_extra import point2_dist_sq_coordinate_formula
from finite_set import FiniteSet, fs_insert, finite_set_empty_contains_eq,
    finite_set_insert_cardinality_is_suc_of_not_contains, finite_set_ext_contains
from data.basic.logic import or_false
from data.finite.finite_set_card import fs_card, fs_card_eq_of_cardinality_is,
    fs_card_empty, fs_card_cardinality_is
from data.finite.finite_set_filter import finite_set_filter, finite_set_filter_contains_eq
from data.finite.finite_set_product import finite_set_product, finite_set_product_contains_eq,
    finite_set_product_contains_pair, finite_set_product_empty_left
from data.finite.finite_set_membership import fs_insert_contains_eq, fs_insert_contains_self

numerals Real
numerals Nat

/// The Erdős unit-distance problem asks how many unordered pairs of points of a
/// finite planar set can be exactly one unit apart.  For a finite set `p` of
/// points of the coordinate plane this file formalizes
///
///     nu(p) = #{ {a, b} subset p : a != b and |a - b| = 1 },
///
/// and records the milestone-M1 statements around it: Erdős's conjectured
/// upper bound `u(n) <= n^(1 + O(1/log log n))` (disproved by OpenAI and
/// Sawin in 2026, who exhibited point sets with `u(n) >= n^(1 + eps)` for
/// infinitely many `n`), the known Spencer–Szemerédi–Trotter upper bound
/// `u(n) = O(n^(4/3))`, and the grid lower bound `u(n^2) >= c n^2`.
///
/// The library works with squared Euclidean distances, so "distance exactly
/// one" is stated as `dist_sq = 1`.  The extremal quantity
/// `u(n) = max { nu(p) : |p| = n }` is a maximum over an infinite family and
/// is not directly expressible; every statement below is therefore given in
/// the equivalent point-set form "for every finite point set `p` with
/// `|p| >= n0`, ...".
///
/// Two small values of `nu` are proved to show the definition is usable:
/// the unit segment has `nu = 1` (so `u(2) = 1`), and the empty set has
/// `nu = 0`.  The four milestone statements (the disproved conjecture, the
/// O(n^(4/3)) upper bound, the 2026 disproof, and the grid lower bound) are
/// recorded as commented-out theorem texts at the end of the file.

/// True when two points are at Euclidean distance exactly one.
define points_unit_distance(a: Point2[Real], b: Point2[Real]) -> Bool {
    a.dist_sq(b) = Real.1
}

/// True of an ordered pair whose components are at unit distance.
define ordered_unit_pair(q: Pair[Point2[Real], Point2[Real]]) -> Bool {
    points_unit_distance(q.first, q.second)
}

/// The ordered pairs of points of `p` at unit distance.
define ordered_unit_pairs(p: FiniteSet[Point2[Real]]) -> FiniteSet[Pair[Point2[Real], Point2[Real]]] {
    finite_set_filter(finite_set_product(p, p), ordered_unit_pair)
}

/// The number of unordered pairs of distinct points of `p` at distance exactly one.
///
/// A point is at squared distance zero from itself, so unit pairs are
/// automatically pairs of distinct points, and each unordered unit pair
/// `{a, b}` is counted twice among the ordered unit pairs, as `(a, b)` and
/// `(b, a)`.  The ordered count is therefore exactly twice `nu(p)`.
define nu(p: FiniteSet[Point2[Real]]) -> Nat {
    fs_card(ordered_unit_pairs(p)).div(Nat.2)
}

/// The unit segment: the two points (0, 0) and (1, 0).
let p0: Point2[Real] = Point2.new(Real.0, Real.0)
let p1: Point2[Real] = Point2.new(Real.1, Real.0)
let unit_segment: FiniteSet[Point2[Real]] =
    fs_insert(fs_insert(FiniteSet.empty[Point2[Real]], p0), p1)

/// The real number three.
let real_three: Real = Real.1 + Real.1 + Real.1

/// The real number four.
let real_four: Real = Real.1 + Real.1 + Real.1 + Real.1

/// The two endpoints of the unit segment are distinct.
theorem unit_segment_points_ne {
    p0 != p1
} by {
    if p0 = p1 {
        point2_ext(p0, p1)
        p0.x = p1.x
        point2_new_x(Real.0, Real.0)
        point2_new_x(Real.1, Real.0)
        p0.x = Real.0
        p1.x = Real.1
        Real.0 = Real.1
        false
    }
    p0 != p1
}

/// The coordinate arithmetic behind the unit-segment distance: (0-1)^2 + (0-0)^2 = 1.
theorem unit_segment_ring_step {
    (Real.0 - Real.1) * (Real.0 - Real.1) + (Real.0 - Real.0) * (Real.0 - Real.0) = Real.1
} by {
    (Real.0 - Real.1) * (Real.0 - Real.1) = Real.1
    (Real.0 - Real.0) * (Real.0 - Real.0) = Real.0
    (Real.0 - Real.1) * (Real.0 - Real.1) + (Real.0 - Real.0) * (Real.0 - Real.0) = Real.1 + Real.0
    Real.1 + Real.0 = Real.1
    (Real.0 - Real.1) * (Real.0 - Real.1) + (Real.0 - Real.0) * (Real.0 - Real.0) = Real.1
}

/// The two endpoints of the unit segment are at unit distance.
theorem unit_segment_endpoints_unit_distance {
    points_unit_distance(p0, p1)
} by {
    point2_dist_sq_coordinate_formula(p0, p1)
    p0.dist_sq(p1) = (p0.x - p1.x) * (p0.x - p1.x) + (p0.y - p1.y) * (p0.y - p1.y)
    point2_new_x(Real.0, Real.0)
    point2_new_y(Real.0, Real.0)
    point2_new_x(Real.1, Real.0)
    point2_new_y(Real.1, Real.0)
    p0.x = Real.0
    p0.y = Real.0
    p1.x = Real.1
    p1.y = Real.0
    p0.dist_sq(p1) = (Real.0 - Real.1) * (Real.0 - Real.1) + (Real.0 - Real.0) * (Real.0 - Real.0)
    unit_segment_ring_step
    p0.dist_sq(p1) = Real.1
    points_unit_distance(p0, p1)
}

/// A member of the unit segment is one of its two endpoints.
theorem unit_segment_contains_iff(x: Point2[Real]) {
    unit_segment.contains(x) = (x = p0 or x = p1)
} by {
    fs_insert_contains_eq(fs_insert(FiniteSet.empty[Point2[Real]], p0), p1, x)
    unit_segment.contains(x) =
        (x = p1 or fs_insert(FiniteSet.empty[Point2[Real]], p0).contains(x))
    fs_insert_contains_eq(FiniteSet.empty[Point2[Real]], p0, x)
    fs_insert(FiniteSet.empty[Point2[Real]], p0).contains(x) = (x = p0 or FiniteSet.empty[Point2[Real]].contains(x))
    finite_set_empty_contains_eq(x)
    FiniteSet.empty[Point2[Real]].contains(x) = false
    unit_segment.contains(x) = (x = p1 or (x = p0 or false))
    unit_segment.contains(x) = (x = p0 or x = p1)
}

/// The ordered unit pairs of the unit segment are exactly (p0, p1) and (p1, p0).
theorem unit_segment_ordered_unit_pairs_eq {
    ordered_unit_pairs(unit_segment) =
        fs_insert(fs_insert(FiniteSet.empty[Pair[Point2[Real], Point2[Real]]], Pair.new(p0, p1)), Pair.new(p1, p0))
} by {
    let left = ordered_unit_pairs(unit_segment)
    let right = fs_insert(fs_insert(FiniteSet.empty[Pair[Point2[Real], Point2[Real]]], Pair.new(p0, p1)), Pair.new(p1, p0))
    forall(q: Pair[Point2[Real], Point2[Real]]) {
        finite_set_filter_contains_eq(finite_set_product(unit_segment, unit_segment), ordered_unit_pair, q)
        left.contains(q) = (finite_set_product(unit_segment, unit_segment).contains(q) and ordered_unit_pair(q))
        finite_set_product_contains_eq(unit_segment, unit_segment, q)
        finite_set_product(unit_segment, unit_segment).contains(q) =
            (unit_segment.contains(q.first) and unit_segment.contains(q.second))
        unit_segment_contains_iff(q.first)
        unit_segment_contains_iff(q.second)
        if left.contains(q) {
            finite_set_product(unit_segment, unit_segment).contains(q)
            ordered_unit_pair(q)
            unit_segment.contains(q.first)
            unit_segment.contains(q.second)
            unit_segment.contains(q.first) = (q.first = p0 or q.first = p1)
            unit_segment.contains(q.second) = (q.second = p0 or q.second = p1)
            if q.first = p0 {
                if q.second = p0 {
                    q.first.dist_sq(q.second) = Real.1
                    p0.dist_sq(p0) = Real.1
                    point2_dist_sq_self(p0)
                    p0.dist_sq(p0) = Real.0
                    false
                }
                if q.second = p1 {
                    pair_ext(q, Pair.new(p0, p1))
                    pair_new_first(p0, p1)
                    pair_new_second(p0, p1)
                    q.first = Pair.new(p0, p1).first
                    q.second = Pair.new(p0, p1).second
                    q = Pair.new(p0, p1)
                    fs_insert_contains_self(fs_insert(FiniteSet.empty[Pair[Point2[Real], Point2[Real]]], Pair.new(p0, p1)), Pair.new(p1, p0))
                    fs_insert(fs_insert(FiniteSet.empty[Pair[Point2[Real], Point2[Real]]], Pair.new(p0, p1)), Pair.new(p1, p0)).contains(Pair.new(p1, p0))
                    fs_insert_contains_eq(fs_insert(FiniteSet.empty[Pair[Point2[Real], Point2[Real]]], Pair.new(p0, p1)), Pair.new(p1, p0), Pair.new(p0, p1))
                    fs_insert(fs_insert(FiniteSet.empty[Pair[Point2[Real], Point2[Real]]], Pair.new(p0, p1)), Pair.new(p1, p0)).contains(Pair.new(p0, p1)) =
                        (Pair.new(p0, p1) = Pair.new(p1, p0) or fs_insert(FiniteSet.empty[Pair[Point2[Real], Point2[Real]]], Pair.new(p0, p1)).contains(Pair.new(p0, p1)))
                    fs_insert_contains_self(FiniteSet.empty[Pair[Point2[Real], Point2[Real]]], Pair.new(p0, p1))
                    fs_insert(FiniteSet.empty[Pair[Point2[Real], Point2[Real]]], Pair.new(p0, p1)).contains(Pair.new(p0, p1))
                    fs_insert(fs_insert(FiniteSet.empty[Pair[Point2[Real], Point2[Real]]], Pair.new(p0, p1)), Pair.new(p1, p0)).contains(Pair.new(p0, p1))
                    right.contains(Pair.new(p0, p1))
                    right.contains(q)
                }
                unit_segment_contains_iff(q.second)
                q.second = p0 or q.second = p1
                right.contains(q)
            }
            if q.first = p1 {
                if q.second = p0 {
                    pair_ext(q, Pair.new(p1, p0))
                    pair_new_first(p1, p0)
                    pair_new_second(p1, p0)
                    q.first = Pair.new(p1, p0).first
                    q.second = Pair.new(p1, p0).second
                    q = Pair.new(p1, p0)
                    fs_insert_contains_self(fs_insert(FiniteSet.empty[Pair[Point2[Real], Point2[Real]]], Pair.new(p0, p1)), Pair.new(p1, p0))
                    fs_insert(fs_insert(FiniteSet.empty[Pair[Point2[Real], Point2[Real]]], Pair.new(p0, p1)), Pair.new(p1, p0)).contains(Pair.new(p1, p0))
                    right.contains(Pair.new(p1, p0))
                    right.contains(q)
                }
                if q.second = p1 {
                    q.first.dist_sq(q.second) = Real.1
                    p1.dist_sq(p1) = Real.1
                    point2_dist_sq_self(p1)
                    p1.dist_sq(p1) = Real.0
                    false
                }
                unit_segment_contains_iff(q.second)
                q.second = p0 or q.second = p1
                right.contains(q)
            }
            q.first = p0 or q.first = p1
            right.contains(q)
        }
        if right.contains(q) {
            fs_insert_contains_eq(fs_insert(FiniteSet.empty[Pair[Point2[Real], Point2[Real]]], Pair.new(p0, p1)), Pair.new(p1, p0), q)
            right.contains(q) = (q = Pair.new(p1, p0) or fs_insert(FiniteSet.empty[Pair[Point2[Real], Point2[Real]]], Pair.new(p0, p1)).contains(q))
            if q = Pair.new(p0, p1) {
                pair_new_first(p0, p1)
                pair_new_second(p0, p1)
                Pair.new(p0, p1).first = p0
                Pair.new(p0, p1).second = p1
                finite_set_product_contains_pair(unit_segment, unit_segment, p0, p1)
                finite_set_product(unit_segment, unit_segment).contains(Pair.new(p0, p1)) =
                    (unit_segment.contains(p0) and unit_segment.contains(p1))
                unit_segment_contains_iff(p0)
                unit_segment_contains_iff(p1)
                unit_segment.contains(p0)
                unit_segment.contains(p1)
                finite_set_product(unit_segment, unit_segment).contains(Pair.new(p0, p1))
                unit_segment_endpoints_unit_distance
                points_unit_distance(p0, p1)
                ordered_unit_pair(Pair.new(p0, p1))
                finite_set_filter_contains_eq(finite_set_product(unit_segment, unit_segment), ordered_unit_pair, Pair.new(p0, p1))
                left.contains(Pair.new(p0, p1)) =
                    (finite_set_product(unit_segment, unit_segment).contains(Pair.new(p0, p1)) and ordered_unit_pair(Pair.new(p0, p1)))
                left.contains(Pair.new(p0, p1))
                q = Pair.new(p0, p1)
                left.contains(q)
            }
            fs_insert_contains_eq(FiniteSet.empty[Pair[Point2[Real], Point2[Real]]], Pair.new(p0, p1), q)
            fs_insert(FiniteSet.empty[Pair[Point2[Real], Point2[Real]]], Pair.new(p0, p1)).contains(q) =
                (q = Pair.new(p0, p1) or FiniteSet.empty[Pair[Point2[Real], Point2[Real]]].contains(q))
            finite_set_empty_contains_eq(q)
            FiniteSet.empty[Pair[Point2[Real], Point2[Real]]].contains(q) = false
            q = Pair.new(p1, p0) or (q = Pair.new(p0, p1) or FiniteSet.empty[Pair[Point2[Real], Point2[Real]]].contains(q))
            q = Pair.new(p1, p0) or (q = Pair.new(p0, p1) or false)
            or_false(q = Pair.new(p0, p1))
            (q = Pair.new(p0, p1) or false) = (q = Pair.new(p0, p1))
            q = Pair.new(p1, p0) or q = Pair.new(p0, p1)
            if q = Pair.new(p1, p0) {
                pair_new_first(p1, p0)
                pair_new_second(p1, p0)
                Pair.new(p1, p0).first = p1
                Pair.new(p1, p0).second = p0
                finite_set_product_contains_pair(unit_segment, unit_segment, p1, p0)
                finite_set_product(unit_segment, unit_segment).contains(Pair.new(p1, p0)) =
                    (unit_segment.contains(p1) and unit_segment.contains(p0))
                unit_segment_contains_iff(p0)
                unit_segment_contains_iff(p1)
                unit_segment.contains(p1)
                unit_segment.contains(p0)
                finite_set_product(unit_segment, unit_segment).contains(Pair.new(p1, p0))
                point2_dist_sq_comm(p0, p1)
                p0.dist_sq(p1) = p1.dist_sq(p0)
                p1.dist_sq(p0) = p0.dist_sq(p1)
                p0.dist_sq(p1) = Real.1
                p1.dist_sq(p0) = Real.1
                points_unit_distance(p1, p0)
                ordered_unit_pair(Pair.new(p1, p0))
                finite_set_filter_contains_eq(finite_set_product(unit_segment, unit_segment), ordered_unit_pair, Pair.new(p1, p0))
                left.contains(Pair.new(p1, p0)) =
                    (finite_set_product(unit_segment, unit_segment).contains(Pair.new(p1, p0)) and ordered_unit_pair(Pair.new(p1, p0)))
                left.contains(Pair.new(p1, p0))
                q = Pair.new(p1, p0)
                left.contains(q)
            }
            left.contains(q)
        }
        left.contains(q) = right.contains(q)
        left.underlying_set.contains(q) = right.underlying_set.contains(q)
    }
    finite_set_ext_contains(left, right)
    left = right
    ordered_unit_pairs(unit_segment) =
        fs_insert(fs_insert(FiniteSet.empty[Pair[Point2[Real], Point2[Real]]], Pair.new(p0, p1)), Pair.new(p1, p0))
}

/// The ordered unit pairs of the unit segment number exactly two.
theorem unit_segment_ordered_unit_pairs_card {
    fs_card(ordered_unit_pairs(unit_segment)) = Nat.2
} by {
    unit_segment_ordered_unit_pairs_eq
    let right = fs_insert(fs_insert(FiniteSet.empty[Pair[Point2[Real], Point2[Real]]], Pair.new(p0, p1)), Pair.new(p1, p0))
    fs_card_empty[Pair[Point2[Real], Point2[Real]]]
    fs_card(FiniteSet.empty[Pair[Point2[Real], Point2[Real]]]) = Nat.0
    fs_card_cardinality_is(FiniteSet.empty[Pair[Point2[Real], Point2[Real]]])
    FiniteSet.empty[Pair[Point2[Real], Point2[Real]]].cardinality_is(fs_card(FiniteSet.empty[Pair[Point2[Real], Point2[Real]]]))
    FiniteSet.empty[Pair[Point2[Real], Point2[Real]]].cardinality_is(Nat.0)
    finite_set_empty_contains_eq(Pair.new(p0, p1))
    not FiniteSet.empty[Pair[Point2[Real], Point2[Real]]].contains(Pair.new(p0, p1))
    finite_set_insert_cardinality_is_suc_of_not_contains(
        FiniteSet.empty[Pair[Point2[Real], Point2[Real]]], Pair.new(p0, p1), Nat.0)
    fs_insert(FiniteSet.empty[Pair[Point2[Real], Point2[Real]]], Pair.new(p0, p1)).cardinality_is(Nat.1)
    pair_ext(Pair.new(p1, p0), Pair.new(p0, p1))
    pair_new_first(p1, p0)
    pair_new_second(p1, p0)
    pair_new_first(p0, p1)
    pair_new_second(p0, p1)
    Pair.new(p1, p0).first = p1
    Pair.new(p1, p0).second = p0
    Pair.new(p0, p1).first = p0
    Pair.new(p0, p1).second = p1
    if Pair.new(p1, p0) = Pair.new(p0, p1) {
        pair_ext(Pair.new(p1, p0), Pair.new(p0, p1))
        Pair.new(p1, p0).first = Pair.new(p0, p1).first
        Pair.new(p1, p0).second = Pair.new(p0, p1).second
        p1 = p0
        unit_segment_points_ne
        false
    }
    Pair.new(p1, p0) != Pair.new(p0, p1)
    fs_insert_contains_eq(FiniteSet.empty[Pair[Point2[Real], Point2[Real]]], Pair.new(p0, p1), Pair.new(p1, p0))
    fs_insert(FiniteSet.empty[Pair[Point2[Real], Point2[Real]]], Pair.new(p0, p1)).contains(Pair.new(p1, p0)) =
        (Pair.new(p1, p0) = Pair.new(p0, p1) or FiniteSet.empty[Pair[Point2[Real], Point2[Real]]].contains(Pair.new(p1, p0)))
    finite_set_empty_contains_eq(Pair.new(p1, p0))
    FiniteSet.empty[Pair[Point2[Real], Point2[Real]]].contains(Pair.new(p1, p0)) = false
    not fs_insert(FiniteSet.empty[Pair[Point2[Real], Point2[Real]]], Pair.new(p0, p1)).contains(Pair.new(p1, p0))
    finite_set_insert_cardinality_is_suc_of_not_contains(
        fs_insert(FiniteSet.empty[Pair[Point2[Real], Point2[Real]]], Pair.new(p0, p1)), Pair.new(p1, p0), Nat.1)
    right.cardinality_is(Nat.2)
    fs_card_eq_of_cardinality_is(right, Nat.2)
    fs_card(right) = Nat.2
    fs_card(ordered_unit_pairs(unit_segment)) = Nat.2
}

/// The unit distance count of the unit segment is one: `u(2) = 1`.
theorem nu_unit_segment_is_one {
    nu(unit_segment) = Nat.1
} by {
    unit_segment_ordered_unit_pairs_card
    fs_card(ordered_unit_pairs(unit_segment)) = Nat.2
    nu(unit_segment) = fs_card(ordered_unit_pairs(unit_segment)).div(Nat.2)
    fs_card(ordered_unit_pairs(unit_segment)).div(Nat.2) = Nat.2.div(Nat.2)
    div_mul(Nat.1, Nat.2)
    Nat.2 != Nat.0
    (Nat.1 * Nat.2).div(Nat.2) = Nat.1
    Nat.1 * Nat.2 = Nat.2
    Nat.2.div(Nat.2) = Nat.1
    nu(unit_segment) = Nat.1
}

/// The unit distance count of the empty set is zero.
theorem nu_empty_is_zero {
    nu(FiniteSet.empty[Point2[Real]]) = Nat.0
} by {
    finite_set_product_empty_left[Point2[Real], Point2[Real]](FiniteSet.empty[Point2[Real]])
    finite_set_product(FiniteSet.empty[Point2[Real]], FiniteSet.empty[Point2[Real]]) =
        FiniteSet.empty[Pair[Point2[Real], Point2[Real]]]
    let oup = ordered_unit_pairs(FiniteSet.empty[Point2[Real]])
    forall(q: Pair[Point2[Real], Point2[Real]]) {
        finite_set_filter_contains_eq(
            finite_set_product(FiniteSet.empty[Point2[Real]], FiniteSet.empty[Point2[Real]]),
            ordered_unit_pair, q)
        oup.contains(q) =
            (finite_set_product(FiniteSet.empty[Point2[Real]], FiniteSet.empty[Point2[Real]]).contains(q)
                and ordered_unit_pair(q))
        finite_set_empty_contains_eq(q)
        FiniteSet.empty[Pair[Point2[Real], Point2[Real]]].contains(q) = false
        oup.contains(q) = false
        oup.contains(q) = FiniteSet.empty[Pair[Point2[Real], Point2[Real]]].contains(q)
        oup.underlying_set.contains(q) =
            FiniteSet.empty[Pair[Point2[Real], Point2[Real]]].underlying_set.contains(q)
    }
    finite_set_ext_contains(oup, FiniteSet.empty[Pair[Point2[Real], Point2[Real]]])
    oup = FiniteSet.empty[Pair[Point2[Real], Point2[Real]]]
    fs_card_empty[Pair[Point2[Real], Point2[Real]]]
    fs_card(FiniteSet.empty[Pair[Point2[Real], Point2[Real]]]) = Nat.0
    fs_card(oup) = Nat.0
    nu(FiniteSet.empty[Point2[Real]]) = fs_card(oup).div(Nat.2)
    fs_card(oup).div(Nat.2) = Nat.0.div(Nat.2)
    zero_div(Nat.2)
    Nat.2 != Nat.0
    Nat.0.div(Nat.2) = Nat.0
    nu(FiniteSet.empty[Point2[Real]]) = Nat.0
}

// ---------------------------------------------------------------------------
// Milestone M1 statements (Erdős unit-distance problem)
//
// The extremal quantity u(n) = max { nu(p) : |p| = n } is a maximum over an
// infinite family of point sets, which this library cannot express directly.
// Every statement below is therefore the equivalent point-set form: a bound on
// u(n) is a bound on nu(p) that holds uniformly for every finite point set p
// of the relevant size.  None of the statements is proved here: (a) is false
// (disproved in 2026), (c) is the deep 2026 disproof, and (b), (d) need
// incidence or construction machinery that is out of scope for milestone M1.
// They are kept as commented-out theorem texts below, ready to be proved or
// cited by later milestones.
// ---------------------------------------------------------------------------

// (a) Erdős's conjectured upper bound u(n) <= n^(1 + O(1/log log n)), i.e.
//     u(n) <= n^(1 + c / log log n) for some constant c and every n >= 3.
//     DISPROVED by OpenAI (announced 2026-05-20) and by Sawin
//     (arXiv:2605.20579): there are infinitely many n with u(n) >= n^(1 + eps)
//     for a fixed eps > 0.
// theorem erdos_unit_distance_conjecture {
//     exists(c: Real) {
//         c > Real.0 and forall(p: FiniteSet[Point2[Real]]) {
//             Nat.3 <= fs_card(p) implies exists(v: Real) {
//                 (from_nat[Real](fs_card(p))).rpow(
//                     Real.1 + c / ((from_nat[Real](fs_card(p))).log.get_or_else(Real.0)).log.get_or_else(Real.0)) =
//                     Option.some(v)
//                 and from_nat[Real](nu(p)) <= v
//             }
//         }
//     }
// } by {
// }

// (b) The known upper bound u(n) = O(n^(4/3)), Spencer–Szemerédi–Trotter 1984.
//     Stated for all sufficiently large point sets, as the asymptotic notation
//     requires.
// theorem erdos_unit_distance_upper_bound {
//     exists(c: Real) {
//         c > Real.0 and exists(n0: Nat) {
//             forall(p: FiniteSet[Point2[Real]]) {
//                 n0 <= fs_card(p) implies exists(v: Real) {
//                     (from_nat[Real](fs_card(p))).rpow(real_four / real_three) = Option.some(v)
//                     and from_nat[Real](nu(p)) <= c * v
//                 }
//             }
//         }
//     }
// } by {
// }

// (c) The disproof, in the exact form of the Lean formalisation of the result:
//     there is delta > 0 such that for every bound N there is a point set of
//     size n >= N with nu(p) >= n^(1 + delta).  Proved by OpenAI / Sawin
//     (2026); unproved here.
// theorem erdos_unit_distance_lower_bound {
//     exists(delta: Real) {
//         delta > Real.0 and forall(n_ub: Nat) {
//             exists(n: Nat) {
//                 n_ub <= n and exists(p: FiniteSet[Point2[Real]]) {
//                     p.cardinality_is(n) and exists(v: Real) {
//                         (from_nat[Real](n)).rpow(Real.1 + delta) = Option.some(v)
//                         and from_nat[Real](nu(p)) >= v
//                     }
//                 }
//             }
//         }
//     }
// } by {
// }

// (d) The grid lower bound: the n by n grid has n^2 points and about 2 n (n-1)
//     unit pairs, so u(n^2) >= c n^2 for every n >= 2.  (The stronger
//     super-polynomial bound u(n) >= n^(1 + c / log log n) of Erdős 1946 comes
//     from lattice points on circles, not from the plain grid.)
// theorem erdos_unit_distance_grid_lower_bound {
//     exists(c: Real) {
//         c > Real.0 and forall(n: Nat) {
//             Nat.2 <= n implies exists(p: FiniteSet[Point2[Real]]) {
//                 p.cardinality_is(n * n) and from_nat[Real](nu(p)) >= c * from_nat[Real](n * n)
//             }
//         }
//     }
// } by {
// }
