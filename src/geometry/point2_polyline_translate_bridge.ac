from ordered_field import OrderedField
from list import List
from geometry.point2 import Point2
from geometry.point2_affine import point2_translate_neg_right
from geometry.point2_polygon import point2_translate_points, point2_translate_points_nil,
    point2_translate_points_cons
from geometry.point2_polygon_edges import point2_polyline_contains,
    point2_polyline_contains_translate_forward

lemma point2_polyline_translate_list_induction[Item](
    pred: List[Item] -> Bool, items: List[Item]
) {
    pred(List.nil[Item]) and
    forall(head: Item, tail: List[Item]) { pred(tail) implies pred(List.cons(head, tail)) }
    implies pred(items)
} by {
    if pred(List.nil[Item]) and
        forall(head: Item, tail: List[Item]) { pred(tail) implies pred(List.cons(head, tail)) } {
        List.induction(pred)
        forall(xs: List[Item]) { pred(xs) }
        pred(items)
    }
}

/// Translating every point by `v` and then by `-v` recovers the original list.
theorem point2_translate_points_neg_right[T: OrderedField](
    points: List[Point2[T]], v: Point2[T]
) {
    point2_translate_points(point2_translate_points(points, v), v.neg) = points
} by {
    define q(xs: List[Point2[T]]) -> Bool {
        point2_translate_points(point2_translate_points(xs, v), v.neg) = xs
    }

    point2_translate_points_nil(v)
    point2_translate_points_nil(v.neg)
    q(List.nil[Point2[T]])

    forall(head: Point2[T], tail: List[Point2[T]]) {
        if q(tail) {
            point2_translate_points_cons(head, tail, v)
            point2_translate_points_cons(head.translate(v), point2_translate_points(tail, v), v.neg)
            point2_translate_neg_right(head, v)
            point2_translate_points(point2_translate_points(tail, v), v.neg) = tail
            point2_translate_points(point2_translate_points(List.cons(head, tail), v), v.neg) =
                List.cons(head, tail)
            q(List.cons(head, tail))
        }
    }

    point2_polyline_translate_list_induction(q, points)
    q(points)
}

/// If a translated point lies on the translated polyline, the original point lies on the original polyline.
theorem point2_polyline_contains_translate_backward[T: OrderedField](
    points: List[Point2[T]], p: Point2[T], v: Point2[T]
) {
    point2_polyline_contains(point2_translate_points(points, v), p.translate(v))
    implies point2_polyline_contains(points, p)
} by {
    if point2_polyline_contains(point2_translate_points(points, v), p.translate(v)) {
        point2_polyline_contains_translate_forward(point2_translate_points(points, v), p.translate(v), v.neg)
        point2_polyline_contains(
            point2_translate_points(point2_translate_points(points, v), v.neg),
            p.translate(v).translate(v.neg)
        )
        point2_translate_points_neg_right(points, v)
        point2_translate_neg_right(p, v)
        point2_polyline_contains(points, p)
    }
}

/// Translating a polyline and its query point preserves polyline membership exactly.
theorem point2_polyline_contains_translate_iff[T: OrderedField](
    points: List[Point2[T]], p: Point2[T], v: Point2[T]
) {
    point2_polyline_contains(point2_translate_points(points, v), p.translate(v)) =
    point2_polyline_contains(points, p)
} by {
    if point2_polyline_contains(points, p) {
        point2_polyline_contains_translate_forward(points, p, v)
        point2_polyline_contains(point2_translate_points(points, v), p.translate(v))
    }
    if point2_polyline_contains(point2_translate_points(points, v), p.translate(v)) {
        point2_polyline_contains_translate_backward(points, p, v)
        point2_polyline_contains(points, p)
    }
}
