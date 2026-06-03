from ordered_field import OrderedField
from list import List
from geometry.point2 import Point2

/// True when every consecutive triple in a chain is a left turn.
define point2_left_turn_chain[T: OrderedField](first: Point2[T], second: Point2[T], rest: List[Point2[T]]) -> Bool {
    match rest {
        List.nil {
            true
        }
        List.cons(third, tail) {
            if first.left_turn(second, third) {
                point2_left_turn_chain(second, third, tail)
            } else {
                false
            }
        }
    }
}

/// True when every consecutive triple in a chain is a right turn.
define point2_right_turn_chain[T: OrderedField](first: Point2[T], second: Point2[T], rest: List[Point2[T]]) -> Bool {
    match rest {
        List.nil {
            true
        }
        List.cons(third, tail) {
            if first.right_turn(second, third) {
                point2_right_turn_chain(second, third, tail)
            } else {
                false
            }
        }
    }
}

/// True when every consecutive triple in a point list is a left turn.
define point2_all_left_turns[T: OrderedField](points: List[Point2[T]]) -> Bool {
    match points {
        List.nil {
            true
        }
        List.cons(first, tail) {
            match tail {
                List.nil {
                    true
                }
                List.cons(second, rest) {
                    point2_left_turn_chain(first, second, rest)
                }
            }
        }
    }
}

/// True when every consecutive triple in a point list is a right turn.
define point2_all_right_turns[T: OrderedField](points: List[Point2[T]]) -> Bool {
    match points {
        List.nil {
            true
        }
        List.cons(first, tail) {
            match tail {
                List.nil {
                    true
                }
                List.cons(second, rest) {
                    point2_right_turn_chain(first, second, rest)
                }
            }
        }
    }
}

/// An empty left-turn chain is vacuously true.
theorem point2_left_turn_chain_nil[T: OrderedField](first: Point2[T], second: Point2[T]) {
    point2_left_turn_chain(first, second, List.nil[Point2[T]])
}

/// A one-triple left-turn chain is the left-turn predicate of that triple.
theorem point2_left_turn_chain_single[T: OrderedField](first: Point2[T], second: Point2[T], third: Point2[T]) {
    point2_left_turn_chain(first, second, List.cons(third, List.nil[Point2[T]])) = first.left_turn(second, third)
} by {
    if first.left_turn(second, third) {
        point2_left_turn_chain(first, second, List.cons(third, List.nil[Point2[T]]))
    } else {
        not point2_left_turn_chain(first, second, List.cons(third, List.nil[Point2[T]]))
    }
}

/// An empty right-turn chain is vacuously true.
theorem point2_right_turn_chain_nil[T: OrderedField](first: Point2[T], second: Point2[T]) {
    point2_right_turn_chain(first, second, List.nil[Point2[T]])
}

/// A one-triple right-turn chain is the right-turn predicate of that triple.
theorem point2_right_turn_chain_single[T: OrderedField](first: Point2[T], second: Point2[T], third: Point2[T]) {
    point2_right_turn_chain(first, second, List.cons(third, List.nil[Point2[T]])) = first.right_turn(second, third)
} by {
    if first.right_turn(second, third) {
        point2_right_turn_chain(first, second, List.cons(third, List.nil[Point2[T]]))
    } else {
        not point2_right_turn_chain(first, second, List.cons(third, List.nil[Point2[T]]))
    }
}

/// Empty point lists have all consecutive triples left-turning.
theorem point2_all_left_turns_nil[T: OrderedField] {
    point2_all_left_turns(List.nil[Point2[T]])
}

/// Singleton point lists have all consecutive triples left-turning.
theorem point2_all_left_turns_single[T: OrderedField](a: Point2[T]) {
    point2_all_left_turns(List.cons(a, List.nil[Point2[T]]))
}

/// Two-point lists have all consecutive triples left-turning.
theorem point2_all_left_turns_pair[T: OrderedField](a: Point2[T], b: Point2[T]) {
    point2_all_left_turns(List.cons(a, List.cons(b, List.nil[Point2[T]])))
}

/// A three-point list is left-turning exactly when its only triple is a left turn.
theorem point2_all_left_turns_triple[T: OrderedField](a: Point2[T], b: Point2[T], c: Point2[T]) {
    point2_all_left_turns(List.cons(a, List.cons(b, List.cons(c, List.nil[Point2[T]])))) = a.left_turn(b, c)
} by {
    point2_left_turn_chain_single(a, b, c)
}

/// Empty point lists have all consecutive triples right-turning.
theorem point2_all_right_turns_nil[T: OrderedField] {
    point2_all_right_turns(List.nil[Point2[T]])
}

/// Singleton point lists have all consecutive triples right-turning.
theorem point2_all_right_turns_single[T: OrderedField](a: Point2[T]) {
    point2_all_right_turns(List.cons(a, List.nil[Point2[T]]))
}

/// Two-point lists have all consecutive triples right-turning.
theorem point2_all_right_turns_pair[T: OrderedField](a: Point2[T], b: Point2[T]) {
    point2_all_right_turns(List.cons(a, List.cons(b, List.nil[Point2[T]])))
}

/// A three-point list is right-turning exactly when its only triple is a right turn.
theorem point2_all_right_turns_triple[T: OrderedField](a: Point2[T], b: Point2[T], c: Point2[T]) {
    point2_all_right_turns(List.cons(a, List.cons(b, List.cons(c, List.nil[Point2[T]])))) = a.right_turn(b, c)
} by {
    point2_right_turn_chain_single(a, b, c)
}
