from ordered_field import OrderedField
from geometry.point2 import Point2
from geometry.point2_circle import point2_on_circle_eq_dist_sq

/// True when four points all lie on the circle with the given center and squared radius.
///
/// Named so that concyclicity below is a two-variable existential rather than one whose body
/// is a four-way conjunction, which is what lets the permutation arguments go through.
define four_on_circle[T: OrderedField](
    center: Point2[T], radius_sq: T, a: Point2[T], b: Point2[T], c: Point2[T], d: Point2[T]
) -> Bool {
    center.on_circle(radius_sq, a) and center.on_circle(radius_sq, b)
        and center.on_circle(radius_sq, c) and center.on_circle(radius_sq, d)
}

/// Each of the four points lies on the circle.
theorem four_on_circle_apply[T: OrderedField](
    center: Point2[T], radius_sq: T, a: Point2[T], b: Point2[T], c: Point2[T], d: Point2[T]
) {
    four_on_circle(center, radius_sq, a, b, c, d)
        implies center.on_circle(radius_sq, a) and center.on_circle(radius_sq, b)
            and center.on_circle(radius_sq, c) and center.on_circle(radius_sq, d)
} by {
    if four_on_circle(center, radius_sq, a, b, c, d) {
        four_on_circle(center, radius_sq, a, b, c, d) =
            (center.on_circle(radius_sq, a) and center.on_circle(radius_sq, b)
                and center.on_circle(radius_sq, c) and center.on_circle(radius_sq, d))
        (center.on_circle(radius_sq, a) and center.on_circle(radius_sq, b)
            and center.on_circle(radius_sq, c) and center.on_circle(radius_sq, d))
    }
}

/// Four points on the circle satisfy the condition.
theorem four_on_circle_intro[T: OrderedField](
    center: Point2[T], radius_sq: T, a: Point2[T], b: Point2[T], c: Point2[T], d: Point2[T]
) {
    center.on_circle(radius_sq, a) and center.on_circle(radius_sq, b)
        and center.on_circle(radius_sq, c) and center.on_circle(radius_sq, d)
        implies four_on_circle(center, radius_sq, a, b, c, d)
} by {
    if center.on_circle(radius_sq, a) and center.on_circle(radius_sq, b)
        and center.on_circle(radius_sq, c) and center.on_circle(radius_sq, d) {
        four_on_circle(center, radius_sq, a, b, c, d) =
            (center.on_circle(radius_sq, a) and center.on_circle(radius_sq, b)
                and center.on_circle(radius_sq, c) and center.on_circle(radius_sq, d))
        four_on_circle(center, radius_sq, a, b, c, d)
    }
}

/// The condition is unchanged by reordering the four points.
///
/// It is a conjunction over them, so any rearrangement of the conjuncts is the same
/// statement. Stated for the two transpositions that generate the rearrangements used below.
theorem four_on_circle_swap_first_two[T: OrderedField](
    center: Point2[T], radius_sq: T, a: Point2[T], b: Point2[T], c: Point2[T], d: Point2[T]
) {
    four_on_circle(center, radius_sq, a, b, c, d)
        implies four_on_circle(center, radius_sq, b, a, c, d)
} by {
    if four_on_circle(center, radius_sq, a, b, c, d) {
        four_on_circle_apply(center, radius_sq, a, b, c, d)
        center.on_circle(radius_sq, a)
        center.on_circle(radius_sq, b)
        center.on_circle(radius_sq, c)
        center.on_circle(radius_sq, d)
        four_on_circle_intro(center, radius_sq, b, a, c, d)
        four_on_circle(center, radius_sq, b, a, c, d)
    }
}

/// Rotating the four points cyclically leaves the condition unchanged.
theorem four_on_circle_rotate[T: OrderedField](
    center: Point2[T], radius_sq: T, a: Point2[T], b: Point2[T], c: Point2[T], d: Point2[T]
) {
    four_on_circle(center, radius_sq, a, b, c, d)
        implies four_on_circle(center, radius_sq, b, c, d, a)
} by {
    if four_on_circle(center, radius_sq, a, b, c, d) {
        four_on_circle_apply(center, radius_sq, a, b, c, d)
        center.on_circle(radius_sq, a)
        center.on_circle(radius_sq, b)
        center.on_circle(radius_sq, c)
        center.on_circle(radius_sq, d)
        four_on_circle_intro(center, radius_sq, b, c, d, a)
        four_on_circle(center, radius_sq, b, c, d, a)
    }
}

/// True when four points lie on a common circle.
///
/// The squared radius is allowed to be zero, so four copies of a single point count as
/// concyclic. That is the convention that makes the no-four-concyclic condition below imply
/// the points are distinct.
define concyclic4[T: OrderedField](
    a: Point2[T], b: Point2[T], c: Point2[T], d: Point2[T]
) -> Bool {
    exists(center: Point2[T], radius_sq: T) {
        four_on_circle(center, radius_sq, a, b, c, d)
    }
}

/// A common circle witnesses concyclicity.
theorem concyclic4_intro[T: OrderedField](
    center: Point2[T], radius_sq: T, a: Point2[T], b: Point2[T], c: Point2[T], d: Point2[T]
) {
    four_on_circle(center, radius_sq, a, b, c, d) implies concyclic4(a, b, c, d)
} by {
    if four_on_circle(center, radius_sq, a, b, c, d) {
        concyclic4(a, b, c, d) = exists(o: Point2[T], r: T) {
            four_on_circle(o, r, a, b, c, d)
        }
        exists(o: Point2[T], r: T) {
            four_on_circle(o, r, a, b, c, d)
        }
        concyclic4(a, b, c, d)
    }
}

/// A common circle can be extracted.
theorem concyclic4_witness[T: OrderedField](
    a: Point2[T], b: Point2[T], c: Point2[T], d: Point2[T]
) {
    concyclic4(a, b, c, d) implies exists(center: Point2[T], radius_sq: T) {
        four_on_circle(center, radius_sq, a, b, c, d)
    }
} by {
    if concyclic4(a, b, c, d) {
        concyclic4(a, b, c, d) = exists(o: Point2[T], r: T) {
            four_on_circle(o, r, a, b, c, d)
        }
        exists(o: Point2[T], r: T) {
            four_on_circle(o, r, a, b, c, d)
        }
    }
}

/// Concyclicity does not depend on the order of the first two points.
theorem concyclic4_swap_first_two[T: OrderedField](
    a: Point2[T], b: Point2[T], c: Point2[T], d: Point2[T]
) {
    concyclic4(a, b, c, d) implies concyclic4(b, a, c, d)
} by {
    if concyclic4(a, b, c, d) {
        concyclic4_witness(a, b, c, d)
        let (center: Point2[T], radius_sq: T) satisfy {
            four_on_circle(center, radius_sq, a, b, c, d)
        }
        four_on_circle_swap_first_two(center, radius_sq, a, b, c, d)
        four_on_circle(center, radius_sq, b, a, c, d)
        concyclic4_intro(center, radius_sq, b, a, c, d)
        concyclic4(b, a, c, d)
    }
}

/// Concyclicity is unchanged by a cyclic rotation of the four points.
///
/// Together with the transposition above this generates every rearrangement, so concyclicity
/// is a property of the four points as an unordered collection.
theorem concyclic4_rotate[T: OrderedField](
    a: Point2[T], b: Point2[T], c: Point2[T], d: Point2[T]
) {
    concyclic4(a, b, c, d) implies concyclic4(b, c, d, a)
} by {
    if concyclic4(a, b, c, d) {
        concyclic4_witness(a, b, c, d)
        let (center: Point2[T], radius_sq: T) satisfy {
            four_on_circle(center, radius_sq, a, b, c, d)
        }
        four_on_circle_rotate(center, radius_sq, a, b, c, d)
        four_on_circle(center, radius_sq, b, c, d, a)
        concyclic4_intro(center, radius_sq, b, c, d, a)
        concyclic4(b, c, d, a)
    }
}

/// Concyclicity does not depend on the order of the last two points.
///
/// Obtained from the two generators: rotate twice, swap, then rotate twice back.
theorem concyclic4_swap_last_two[T: OrderedField](
    a: Point2[T], b: Point2[T], c: Point2[T], d: Point2[T]
) {
    concyclic4(a, b, c, d) implies concyclic4(a, b, d, c)
} by {
    if concyclic4(a, b, c, d) {
        concyclic4_rotate(a, b, c, d)
        concyclic4(b, c, d, a)
        concyclic4_rotate(b, c, d, a)
        concyclic4(c, d, a, b)
        concyclic4_swap_first_two(c, d, a, b)
        concyclic4(d, c, a, b)
        concyclic4_rotate(d, c, a, b)
        concyclic4(c, a, b, d)
        concyclic4_rotate(c, a, b, d)
        concyclic4(a, b, d, c)
    }
}

/// Any four points on a circle around a common center are concyclic.
theorem concyclic4_of_equal_distances[T: OrderedField](
    center: Point2[T], a: Point2[T], b: Point2[T], c: Point2[T], d: Point2[T]
) {
    a.dist_sq(center) = b.dist_sq(center) and a.dist_sq(center) = c.dist_sq(center)
        and a.dist_sq(center) = d.dist_sq(center)
        implies concyclic4(a, b, c, d)
} by {
    if a.dist_sq(center) = b.dist_sq(center) and a.dist_sq(center) = c.dist_sq(center)
        and a.dist_sq(center) = d.dist_sq(center) {
        point2_on_circle_eq_dist_sq(center, a.dist_sq(center), a)
        center.on_circle(a.dist_sq(center), a)
        point2_on_circle_eq_dist_sq(center, a.dist_sq(center), b)
        center.on_circle(a.dist_sq(center), b)
        point2_on_circle_eq_dist_sq(center, a.dist_sq(center), c)
        center.on_circle(a.dist_sq(center), c)
        point2_on_circle_eq_dist_sq(center, a.dist_sq(center), d)
        center.on_circle(a.dist_sq(center), d)
        four_on_circle_intro(center, a.dist_sq(center), a, b, c, d)
        four_on_circle(center, a.dist_sq(center), a, b, c, d)
        concyclic4_intro(center, a.dist_sq(center), a, b, c, d)
        concyclic4(a, b, c, d)
    }
}

/// Four points on a common circle have equal squared distances to its center.
theorem concyclic4_distances_equal[T: OrderedField](
    center: Point2[T], radius_sq: T, a: Point2[T], b: Point2[T], c: Point2[T], d: Point2[T]
) {
    four_on_circle(center, radius_sq, a, b, c, d)
        implies a.dist_sq(center) = radius_sq and b.dist_sq(center) = radius_sq
            and c.dist_sq(center) = radius_sq and d.dist_sq(center) = radius_sq
} by {
    if four_on_circle(center, radius_sq, a, b, c, d) {
        four_on_circle_apply(center, radius_sq, a, b, c, d)
        center.on_circle(radius_sq, a)
        point2_on_circle_eq_dist_sq(center, radius_sq, a)
        a.dist_sq(center) = radius_sq
        center.on_circle(radius_sq, b)
        point2_on_circle_eq_dist_sq(center, radius_sq, b)
        b.dist_sq(center) = radius_sq
        center.on_circle(radius_sq, c)
        point2_on_circle_eq_dist_sq(center, radius_sq, c)
        c.dist_sq(center) = radius_sq
        center.on_circle(radius_sq, d)
        point2_on_circle_eq_dist_sq(center, radius_sq, d)
        d.dist_sq(center) = radius_sq
        (a.dist_sq(center) = radius_sq and b.dist_sq(center) = radius_sq
            and c.dist_sq(center) = radius_sq and d.dist_sq(center) = radius_sq)
    }
}
