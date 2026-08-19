from algebra.add_comm_monoid import AddCommMonoid
from algebra.add_comm_group import AddCommGroup
from comm_ring import CommRing
from ordered_field import OrderedField
from real import Real
from data.basic.witness import choose_or_default

numerals Real

/// A two-dimensional point with coordinates in `T`.
structure Point2[T] {
    /// The x-coordinate.
    x: T

    /// The y-coordinate.
    y: T
}

/// The origin in a coordinate plane with additive identity.
let point2_zero[T: AddCommMonoid]: Point2[T] = Point2.new(T.0, T.0)

attributes Point2[T: AddCommMonoid] {
    /// The componentwise sum of two points.
    define add(self, other: Point2[T]) -> Point2[T] {
        Point2.new(self.x + other.x, self.y + other.y)
    }
}

attributes Point2[T: AddCommGroup] {
    /// The componentwise additive inverse of a point.
    define neg(self) -> Point2[T] {
        Point2.new(-self.x, -self.y)
    }

    /// The componentwise difference of two points.
    define sub(self, other: Point2[T]) -> Point2[T] {
        Point2.new(self.x - other.x, self.y - other.y)
    }

    /// Translation by a coordinate displacement.
    define translate(self, displacement: Point2[T]) -> Point2[T] {
        self.add(displacement)
    }
}

attributes Point2[T: CommRing] {
    /// Scalar multiplication of both coordinates.
    define smul(self, scalar: T) -> Point2[T] {
        Point2.new(scalar * self.x, scalar * self.y)
    }

    /// The dot product of two coordinate points.
    define dot(self, other: Point2[T]) -> T {
        self.x * other.x + self.y * other.y
    }

    /// The signed two-dimensional cross product determinant.
    define cross(self, other: Point2[T]) -> T {
        self.x * other.y - self.y * other.x
    }

    /// The squared coordinate norm.
    define norm_sq(self) -> T {
        self.dot(self)
    }

    /// The squared coordinate distance to another point.
    define dist_sq(self, other: Point2[T]) -> T {
        self.sub(other).norm_sq
    }

    /// The point with parameter `t` on the directed line to `other`.
    define param_line(self, other: Point2[T], t: T) -> Point2[T] {
        self.add(other.sub(self).smul(t))
    }

    /// The signed orientation determinant of three points.
    define orientation(self, b: Point2[T], c: Point2[T]) -> T {
        b.sub(self).cross(c.sub(self))
    }
}

attributes Point2[T: OrderedField] {
    /// True when `p` lies on the circle with this center and squared radius.
    define on_circle(self, radius_sq: T, p: Point2[T]) -> Bool {
        p.dist_sq(self) = radius_sq
    }

    /// The signed doubled area of an oriented triangle.
    define triangle_area2(self, b: Point2[T], c: Point2[T]) -> T {
        self.orientation(b, c)
    }
}

/// The two middle terms in a four-term sum may be exchanged.
theorem point2_add_pair_rearrange[T: AddCommGroup](a: T, b: T, c: T, d: T) {
    (a + b) + (c + d) = (a + c) + (b + d)
}

/// The squared norm of a sum expands by the dot product.
theorem point2_norm_sq_add_expansion[T: CommRing](a: Point2[T], b: Point2[T]) {
    a.add(b).norm_sq = a.norm_sq + b.norm_sq + a.dot(b) + a.dot(b)
}

/// Circle membership is equality of squared distance to the center.
theorem point2_on_circle_eq_dist_sq[T: OrderedField](center: Point2[T], radius_sq: T, p: Point2[T]) {
    center.on_circle(radius_sq, p) = (p.dist_sq(center) = radius_sq)
}

/// Intersecting parameterized chords of one circle have equal products of squared segment lengths.
theorem point2_intersecting_chords_dist_sq_product_eq[T: OrderedField](
    center: Point2[T], radius_sq: T,
    a: Point2[T], b: Point2[T], c: Point2[T], d: Point2[T], p: Point2[T],
    u: T, v: T
) {
    center.on_circle(radius_sq, a) and
    center.on_circle(radius_sq, b) and
    center.on_circle(radius_sq, c) and
    center.on_circle(radius_sq, d) and
    p = a.param_line(b, u) and
    p = c.param_line(d, v)
    implies
    a.dist_sq(p) * b.dist_sq(p) = c.dist_sq(p) * d.dist_sq(p)
}

/// The product of the four Heron linear factors.
define heron_linear_product[T: CommRing](x: T, y: T, z: T) -> T {
    (x + y + z) * (-x + y + z) * (x - y + z) * (x + y - z)
}

/// Heron's squared identity for a triangle in the coordinate plane.
theorem point2_heron_triangle_squared[T: OrderedField](a: Point2[T], b: Point2[T], c: Point2[T], x: T, y: T, z: T) {
    x * x = c.sub(b).norm_sq and y * y = c.sub(a).norm_sq and z * z = b.sub(a).norm_sq implies
        heron_linear_product(x, y, z) =
        (a.triangle_area2(b, c) + a.triangle_area2(b, c)) *
        (a.triangle_area2(b, c) + a.triangle_area2(b, c))
}

// The law of cosines in squared-distance form, for top100 wrappers.

/// Law of cosines in squared-distance / dot-product form at vertex `a`.
theorem point2_law_of_cosines_dist_sq[T: CommRing](a: Point2[T], b: Point2[T], c: Point2[T]) {
    b.dist_sq(c) =
        a.dist_sq(b) + a.dist_sq(c) -
        b.sub(a).dot(c.sub(a)) - b.sub(a).dot(c.sub(a))
}

// Angle infrastructure (geometry/point2_angle.ac)

/// The Euclidean length of a point: the nonnegative square root of its
/// squared norm (zero outside the nonnegative domain).
define point2_norm(p: Point2[Real]) -> Real {
    match (p.norm_sq).sqrt {
        Option.some(y) {
            y
        }
        Option.none {
            Real.0
        }
    }
}

/// The cosine of the angle between two vectors: the dot product over the
/// product of the lengths.
define cos_angle(u: Point2[Real], v: Point2[Real]) -> Real {
    u.dot(v) / (point2_norm(u) * point2_norm(v))
}

/// Cauchy–Schwarz in squared form for the point dot product: the square of
/// the dot product is at most the product of the squared norms.
theorem point2_cauchy_schwarz_sq[T: OrderedField](u: Point2[T], v: Point2[T]) {
    u.dot(v) * u.dot(v) <= u.norm_sq * v.norm_sq
}

/// The absolute value of the dot product is at most the product of the
/// lengths of the two vectors.
theorem point2_cauchy_schwarz_abs(u: Point2[Real], v: Point2[Real]) {
    (u.dot(v)).abs <= point2_norm(u) * point2_norm(v)
}

/// Cauchy–Schwarz for the point dot product: the dot product is at most the
/// product of the lengths of the two vectors.
theorem point2_cauchy_schwarz(u: Point2[Real], v: Point2[Real]) {
    u.dot(v) <= point2_norm(u) * point2_norm(v)
}

/// The cosine of the angle between two nonzero vectors is at most one.
theorem cos_angle_le_one(u: Point2[Real], v: Point2[Real]) {
    u != point2_zero[Real] and v != point2_zero[Real] implies cos_angle(u, v) <= Real.1
}

/// The cosine of the angle between two nonzero vectors is at least negative
/// one.
theorem cos_angle_ge_neg_one(u: Point2[Real], v: Point2[Real]) {
    u != point2_zero[Real] and v != point2_zero[Real] implies -Real.1 <= cos_angle(u, v)
}

/// The cosine of the angle between two nonzero vectors lies between negative
/// one and one.
theorem cos_angle_bounds(u: Point2[Real], v: Point2[Real]) {
    u != point2_zero[Real] and v != point2_zero[Real] implies
    -Real.1 <= cos_angle(u, v) and cos_angle(u, v) <= Real.1
}

/// The law of cosines in angle-and-side form at vertex `a`: for a nondegenerate
/// triangle with `u = b - a` and `v = c - a`,
/// `|b-c|² = |b-a|² + |c-a|² − 2|b-a||c-a| (∠BAC).cos`.
theorem point2_law_of_cosines_angle(a: Point2[Real], b: Point2[Real], c: Point2[Real]) {
    b != a and c != a implies
    b.dist_sq(c) =
        a.dist_sq(b) + a.dist_sq(c) -
        (Real.1 + Real.1) * (point2_norm(b.sub(a)) * point2_norm(c.sub(a)) *
            cos_angle(b.sub(a), c.sub(a)))
}

// The law of sines (geometry/point2_angle.ac)

/// The sine of the angle between two vectors: the absolute value of the cross
/// product over the product of the lengths.
define sin_angle(u: Point2[Real], v: Point2[Real]) -> Real {
    (u.cross(v)).abs / (point2_norm(u) * point2_norm(v))
}

/// The absolute value of the cross product is at most the product of the
/// lengths of the two vectors.
theorem point2_cauchy_schwarz_cross_abs(u: Point2[Real], v: Point2[Real]) {
    (u.cross(v)).abs <= point2_norm(u) * point2_norm(v)
}

/// The product of the lengths times the sine of the angle is the absolute
/// value of the cross product, for nonzero vectors.
theorem point2_cross_eq_norm_mul_sin_angle(u: Point2[Real], v: Point2[Real]) {
    u != point2_zero[Real] and v != point2_zero[Real] implies
    point2_norm(u) * point2_norm(v) * sin_angle(u, v) = (u.cross(v)).abs
}

/// The sine of the angle between two nonzero vectors is nonnegative.
theorem sin_angle_nonneg(u: Point2[Real], v: Point2[Real]) {
    u != point2_zero[Real] and v != point2_zero[Real] implies Real.0 <= sin_angle(u, v)
}

/// The sine of the angle between two nonzero vectors is at most one.
theorem sin_angle_le_one(u: Point2[Real], v: Point2[Real]) {
    u != point2_zero[Real] and v != point2_zero[Real] implies sin_angle(u, v) <= Real.1
}

/// The doubled area of a triangle is the absolute value of the cross product
/// of the two side vectors meeting at any vertex.
theorem point2_area2_cross_vertex_invariant(a: Point2[Real], b: Point2[Real], c: Point2[Real]) {
    (c.sub(b).cross(a.sub(b))).abs = (b.sub(a).cross(c.sub(a))).abs
}

/// The law of sines in sine form: for a nondegenerate triangle, a side times
/// the sine of the opposite angle is the same for each side.  With
/// `a = |b-c|` the side opposite `A = ∠BAC` and `b = |c-a|` the side opposite
/// `B = ∠CBA`, the identity `a·(B).sin = b·(A).sin` holds.
theorem point2_law_of_sines_sine_form(a: Point2[Real], b: Point2[Real], c: Point2[Real]) {
    b != a and c != a and c != b implies
    point2_norm(c.sub(b)) * sin_angle(a.sub(b), c.sub(b)) =
    point2_norm(c.sub(a)) * sin_angle(b.sub(a), c.sub(a))
}

/// Rotating the vertex of the side vectors preserves their cross product:
/// `(c-b)×(a-b) = (b-a)×(c-a)`.
theorem point2_cross_vertex_rotate(a: Point2[Real], b: Point2[Real], c: Point2[Real]) {
    c.sub(b).cross(a.sub(b)) = b.sub(a).cross(c.sub(a))
}

/// The sine of the angle between two nonzero, nonparallel vectors is nonzero.
theorem point2_sin_angle_ne_zero(u: Point2[Real], v: Point2[Real]) {
    u != point2_zero[Real] and v != point2_zero[Real] and u.cross(v) != Real.0 implies
    sin_angle(u, v) != Real.0
}

/// The law of sines: for a nondegenerate triangle, a side over the sine of
/// the opposite angle is the same for each side.  With `a = |b-c|` the side
/// opposite `A = ∠BAC` and `b = |c-a|` the side opposite `B = ∠CBA`,
/// `a / (A).sin = b / (B).sin`.
theorem point2_law_of_sines(a: Point2[Real], b: Point2[Real], c: Point2[Real]) {
    b != a and c != a and c != b and b.sub(a).cross(c.sub(a)) != Real.0 implies
    point2_norm(c.sub(b)) / sin_angle(b.sub(a), c.sub(a)) =
    point2_norm(c.sub(a)) / sin_angle(a.sub(b), c.sub(b))
}
