/// Real arithmetic for the 3-4-5 right-triangle examples.
///
/// The digits `Real.2` and above are not defined in the library, so each
/// constant is built from `Real.1` by addition.  This module stays free of
/// geometry imports so that the arithmetic goals below are closed directly;
/// the geometry layer imports the facts it needs from here.

from real import Real, mul_inverse, mul_div_cancel, mul_one_over, mul_div_left,
    pos_gt_zero, gt_zero_imp_pos, mul_pos_pos, lt_add_pos, lt_trans,
    from_nat_is_from_rat
from order import lt_imp_ne, lt_imp_lte

numerals Real

/// The real number two.
let real_two: Real = Real.1 + Real.1

/// The real number three.
let real_three: Real = real_two + Real.1

/// The real number four.
let real_four: Real = real_three + Real.1

/// The real number five.
let real_five: Real = real_four + Real.1

/// The real number six.
let real_six: Real = real_five + Real.1

/// The real number eight.
let real_eight: Real = real_six + real_two

/// The real number nine.
let real_nine: Real = real_six + real_three

/// The real number twelve.
let real_twelve: Real = real_six + real_six

/// The real number sixteen.
let real_sixteen: Real = real_twelve + real_four

/// The real number twenty-five.
let real_twentyfive: Real = real_sixteen + real_nine

/// The real number eighteen.
let real_eighteen: Real = real_twelve + real_six

/// The real number thirty-six.
let real_thirty_six: Real = real_six * real_six

/// Six is positive.
theorem real_six_pos {
    Real.0 < real_six
} by {
    Real.1.is_positive
    pos_gt_zero(Real.1)
    Real.0 < Real.1
    Real.1.is_positive
    lt_add_pos(Real.1, Real.1)
    Real.1 < Real.1 + Real.1
    Real.1 < real_two
    lt_trans(Real.0, Real.1, real_two)
    Real.0 < real_two
    Real.1.is_positive
    lt_add_pos(real_two, Real.1)
    real_two < real_two + Real.1
    real_two < real_three
    lt_trans(Real.0, real_two, real_three)
    Real.0 < real_three
    Real.1.is_positive
    lt_add_pos(real_three, Real.1)
    real_three < real_three + Real.1
    real_three < real_four
    lt_trans(Real.0, real_three, real_four)
    Real.0 < real_four
    Real.1.is_positive
    lt_add_pos(real_four, Real.1)
    real_four < real_four + Real.1
    real_four < real_five
    lt_trans(Real.0, real_four, real_five)
    Real.0 < real_five
    Real.1.is_positive
    lt_add_pos(real_five, Real.1)
    real_five < real_five + Real.1
    real_five < real_six
    lt_trans(Real.0, real_five, real_six)
    Real.0 < real_six
}

/// Six is nonnegative.
theorem real_six_nonneg {
    real_six >= Real.0
} by {
    real_six_pos
    Real.0 < real_six
    lt_imp_lte(Real.0, real_six)
    Real.0 <= real_six
}

/// Two is nonzero.
theorem real_two_ne_zero {
    real_two != Real.0
} by { }

/// Two plus three is five.
theorem real_two_add_three_is_five {
    real_two + real_three = real_five
} by {
    real_three = real_two + Real.1
    real_two + real_three = real_two + (real_two + Real.1)
    real_two + (real_two + Real.1) = (real_two + real_two) + Real.1
    real_two + real_two = real_four
    (real_two + real_two) + Real.1 = real_four + Real.1
    real_four + Real.1 = real_five
    real_two + real_three = real_five
}

/// Three plus three is six.
theorem real_three_add_three_is_six {
    real_three + real_three = real_six
} by {
    real_three = real_two + Real.1
    real_three + real_three = (real_two + Real.1) + (real_two + Real.1)
    (real_two + Real.1) + (real_two + Real.1) = (real_two + real_two) + (Real.1 + Real.1)
    real_two + real_two = real_four
    Real.1 + Real.1 = real_two
    (real_two + real_two) + (Real.1 + Real.1) = real_four + real_two
    real_four + real_two = real_six
    real_three + real_three = real_six
}

/// Four plus five is nine.
theorem real_four_add_five_is_nine {
    real_four + real_five = real_nine
} by {
    real_two_add_three_is_five
    real_two + real_three = real_five
    real_six = real_four + real_two
    real_nine = real_six + real_three
    real_nine = (real_four + real_two) + real_three
    (real_four + real_two) + real_three = real_four + (real_two + real_three)
    real_nine = real_four + (real_two + real_three)
    real_nine = real_four + real_five
    real_four + real_five = real_nine
}

/// Nine plus three is twelve.
theorem real_nine_add_three_is_twelve {
    real_nine + real_three = real_twelve
} by { }

/// Three plus four plus five is twelve.
theorem real_three_add_four_add_five_is_twelve {
    real_three + real_four + real_five = real_twelve
} by {
    real_four_add_five_is_nine
    real_four + real_five = real_nine
    real_three + real_four + real_five = real_three + (real_four + real_five)
    real_three + (real_four + real_five) = real_three + real_nine
    real_nine_add_three_is_twelve
    real_nine + real_three = real_twelve
    real_three + real_nine = real_twelve
    real_three + real_four + real_five = real_twelve
}

/// Six minus three is three.
theorem real_six_sub_three_is_three {
    real_six - real_three = real_three
} by {
    real_three_add_three_is_six
    real_three + real_three = real_six
    real_six = real_three + real_three
    real_six - real_three = (real_three + real_three) - real_three
    (real_three + real_three) - real_three = real_three
}

/// Six minus four is two.
theorem real_six_sub_four_is_two {
    real_six - real_four = real_two
} by {
    real_six = real_four + real_two
    real_six - real_four = (real_four + real_two) - real_four
    (real_four + real_two) - real_four = real_two
}

/// Six minus five is one.
theorem real_six_sub_five_is_one {
    real_six - real_five = Real.1
} by { }

/// Two times two is four.
theorem real_two_mul_two_is_four {
    real_two * real_two = real_four
} by {
    real_two = Real.1 + Real.1
    real_two * real_two = (Real.1 + Real.1) * (Real.1 + Real.1)
    (Real.1 + Real.1) * (Real.1 + Real.1) =
        Real.1 * (Real.1 + Real.1) + Real.1 * (Real.1 + Real.1)
    Real.1 * (Real.1 + Real.1) = Real.1 + Real.1
    Real.1 * (Real.1 + Real.1) + Real.1 * (Real.1 + Real.1) =
        (Real.1 + Real.1) + (Real.1 + Real.1)
    (Real.1 + Real.1) + (Real.1 + Real.1) = real_two + real_two
    real_two + real_two = real_four
    real_two * real_two = real_four
}

/// Three times two is six.
theorem real_three_mul_two_is_six {
    real_three * real_two = real_six
} by { }

/// Four times two is eight.
theorem real_four_mul_two_is_eight {
    real_four * real_two = real_eight
} by { }

/// Six times two is twelve.
theorem real_six_mul_two_is_twelve {
    real_six * real_two = real_twelve
} by { }

/// Three squared is nine.
theorem real_three_sq_is_nine {
    real_three * real_three = real_nine
} by { }

/// Four squared is sixteen.
theorem real_four_sq_is_sixteen {
    real_four * real_four = real_sixteen
} by {
    real_four = real_three + Real.1
    real_four * real_four = (real_three + Real.1) * (real_three + Real.1)
    (real_three + Real.1) * (real_three + Real.1) =
        real_three * (real_three + Real.1) + Real.1 * (real_three + Real.1)
    real_three * (real_three + Real.1) = real_three * real_three + real_three * Real.1
    Real.1 * (real_three + Real.1) = real_three + Real.1
    real_three * real_three + real_three * Real.1 + (real_three + Real.1) =
        real_nine + real_three + real_four
    real_nine_add_three_is_twelve
    real_nine + real_three = real_twelve
    real_nine + real_three + real_four = real_twelve + real_four
    real_twelve + real_four = real_sixteen
    real_four * real_four = real_sixteen
}

/// Six times three is eighteen.
theorem real_six_mul_three_is_eighteen {
    real_six * real_three = real_eighteen
} by { }

/// Six times six is thirty-six.
theorem real_six_mul_six_is_thirty_six {
    real_six * real_six = real_thirty_six
} by { }

/// Three times four is twelve.
theorem real_three_mul_four_is_twelve {
    real_three * real_four = real_twelve
} by {
    real_four = real_three + Real.1
    real_three * real_four = real_three * (real_three + Real.1)
    real_three * (real_three + Real.1) = real_three * real_three + real_three * Real.1
    real_three_sq_is_nine
    real_three * real_three = real_nine
    real_three * Real.1 = real_three
    real_three * real_three + real_three * Real.1 = real_nine + real_three
    real_nine_add_three_is_twelve
    real_nine + real_three = real_twelve
    real_three * real_four = real_twelve
}

/// Eighteen times two is thirty-six.
theorem real_eighteen_mul_two_is_thirty_six {
    real_eighteen * real_two = real_thirty_six
} by {
    real_eighteen = real_six * real_three
    real_eighteen * real_two = (real_six * real_three) * real_two
    (real_six * real_three) * real_two = real_six * (real_three * real_two)
    real_three_mul_two_is_six
    real_three * real_two = real_six
    real_six * (real_three * real_two) = real_six * real_six
    real_six * real_six = real_thirty_six
    real_eighteen * real_two = real_thirty_six
}

/// Five squared is twenty-five.
theorem real_five_sq_is_twentyfive {
    real_five * real_five = real_twentyfive
} by {
    real_five = real_four + Real.1
    real_five * real_five = (real_four + Real.1) * (real_four + Real.1)
    (real_four + Real.1) * (real_four + Real.1) =
        real_four * (real_four + Real.1) + Real.1 * (real_four + Real.1)
    real_four * (real_four + Real.1) = real_four * real_four + real_four * Real.1
    Real.1 * (real_four + Real.1) = real_four + Real.1
    real_four * real_four + real_four * Real.1 + (real_four + Real.1) =
        real_sixteen + real_four + real_five
    real_four_add_five_is_nine
    real_four + real_five = real_nine
    real_sixteen + real_four + real_five = real_sixteen + real_nine
    real_sixteen + real_nine = real_twentyfive
    real_five * real_five = real_twentyfive
}

/// Nine plus sixteen is twenty-five.
theorem real_nine_add_sixteen_is_twentyfive {
    real_nine + real_sixteen = real_twentyfive
} by { }

/// Six times three times two times one is thirty-six.
theorem real_six_mul_three_mul_two_mul_one_is_thirty_six {
    real_six * real_three * real_two * Real.1 = real_thirty_six
} by { }

/// Division distributes over addition on the left.
theorem div_add_distrib_local(a: Real, b: Real, c: Real) {
    c != Real.0 implies (a + b) / c = a / c + b / c
} by {
    if c != Real.0 {
        (a + b) / c = (a + b) * c.inverse
        (a + b) * c.inverse = a * c.inverse + b * c.inverse
        a / c = a * c.inverse
        b / c = b * c.inverse
        (a + b) / c = a / c + b / c
    }
}

/// Two divided by two is one.
theorem real_two_div_two {
    real_two / real_two = Real.1
} by {
    real_two / real_two = real_two * real_two.inverse
    mul_inverse(real_two)
    real_two * real_two.inverse = Real.1
    real_two / real_two = Real.1
}

/// Four divided by two is two.
theorem real_four_div_two {
    real_four / real_two = real_two
} by {
    real_four = real_two + real_two
    real_four / real_two = (real_two + real_two) / real_two
    div_add_distrib_local(real_two, real_two, real_two)
    real_two_ne_zero
    (real_two + real_two) / real_two = real_two / real_two + real_two / real_two
    real_two_div_two
    real_two / real_two = Real.1
    real_two / real_two + real_two / real_two = Real.1 + Real.1
    Real.1 + Real.1 = real_two
    real_four / real_two = real_two
}

/// Six divided by two is three.
theorem real_six_div_two {
    real_six / real_two = real_three
} by {
    real_six = real_four + real_two
    real_six / real_two = (real_four + real_two) / real_two
    div_add_distrib_local(real_four, real_two, real_two)
    real_two_ne_zero
    (real_four + real_two) / real_two = real_four / real_two + real_two / real_two
    real_four_div_two
    real_four / real_two = real_two
    real_two_div_two
    real_two / real_two = Real.1
    real_four / real_two + real_two / real_two = real_two + Real.1
    real_two + Real.1 = real_three
    real_six / real_two = real_three
}

/// Twelve divided by two is six.
theorem real_twelve_div_two_is_six {
    real_twelve / real_two = real_six
} by {
    real_twelve = real_six + real_six
    real_twelve / real_two = (real_six + real_six) / real_two
    div_add_distrib_local(real_six, real_six, real_two)
    real_two_ne_zero
    (real_six + real_six) / real_two = real_six / real_two + real_six / real_two
    real_six_div_two
    real_six / real_two = real_three
    real_six / real_two + real_six / real_two = real_three + real_three
    real_three_add_three_is_six
    real_three + real_three = real_six
    real_twelve / real_two = real_six
}
