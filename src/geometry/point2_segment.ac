from ordered_field import OrderedField
from geometry.point2 import Point2
from geometry.point2_orientation import point2_collinear_same_first,
    point2_collinear_same_second, point2_collinear_same_third,
    point2_collinear_swap_first_second
from geometry.point2_affine import point2_param_line_zero,
    point2_param_line_one, point2_param_line_collinear,
    point2_param_line_translate, point2_param_line_reverse,
    point2_translate_neg_right, point2_collinear_translate

attributes Point2[T: OrderedField] {
    /// True when `p` lies on the line through `self` and `b`.
    define on_line(self, b: Point2[T], p: Point2[T]) -> Bool {
        self.collinear(b, p)
    }

    /// True when `p` lies on the closed segment from `self` to `b`.
    define on_segment(self, b: Point2[T], p: Point2[T]) -> Bool {
        exists(t: T) {
            T.0 <= t and t <= T.1 and p = self.param_line(b, t)
        }
    }

    /// True when `p` lies between `self` and `b`, inclusive.
    define between(self, p: Point2[T], b: Point2[T]) -> Bool {
        self.on_segment(b, p)
    }
}

/// If `t <= 1`, then `1 - t` is nonnegative.
theorem point2_unit_sub_nonneg[T: OrderedField](t: T) {
    t <= T.1 implies T.0 <= T.1 - t
} by {
    if t <= T.1 {
        t + -t <= T.1 + -t
        T.0 <= T.1 - t
    }
}

/// If `t` is nonnegative, then `1 - t <= 1`.
theorem point2_unit_sub_lte_one[T: OrderedField](t: T) {
    T.0 <= t implies T.1 - t <= T.1
} by {
    if T.0 <= t {
        T.1 + -t <= T.1 + T.0
        T.1 - t <= T.1
    }
}

/// If a parameter is in `[0, 1]`, so is its reversal `1 - t`.
theorem point2_unit_sub_bounds[T: OrderedField](t: T) {
    T.0 <= t and t <= T.1 implies T.0 <= T.1 - t and T.1 - t <= T.1
} by {
    if T.0 <= t and t <= T.1 {
        point2_unit_sub_nonneg(t)
        point2_unit_sub_lte_one(t)
        T.0 <= T.1 - t and T.1 - t <= T.1
    }
}

/// Line membership is collinearity by definition.
theorem point2_on_line_iff_collinear[T: OrderedField](a: Point2[T], b: Point2[T], p: Point2[T]) {
    a.on_line(b, p) = a.collinear(b, p)
}

/// The first defining point lies on the line.
theorem point2_on_line_start[T: OrderedField](a: Point2[T], b: Point2[T]) {
    a.on_line(b, a)
} by {
    point2_collinear_same_second(a, b)
}

/// The second defining point lies on the line.
theorem point2_on_line_end[T: OrderedField](a: Point2[T], b: Point2[T]) {
    a.on_line(b, b)
} by {
    point2_collinear_same_third(a, b)
}

/// Every point lies on the degenerate line through a point and itself.
theorem point2_on_line_degenerate[T: OrderedField](a: Point2[T], p: Point2[T]) {
    a.on_line(a, p)
} by {
    point2_collinear_same_first(a, p)
}

/// Line membership is symmetric in the two defining points.
theorem point2_on_line_swap[T: OrderedField](a: Point2[T], b: Point2[T], p: Point2[T]) {
    a.on_line(b, p) = b.on_line(a, p)
} by {
    point2_collinear_swap_first_second(a, b, p)
}

/// Translating the whole line configuration preserves line membership.
theorem point2_on_line_translate[T: OrderedField](a: Point2[T], b: Point2[T], p: Point2[T], v: Point2[T]) {
    a.translate(v).on_line(b.translate(v), p.translate(v)) = a.on_line(b, p)
} by {
    point2_collinear_translate(a, b, p, v)
}

/// A parameter-line point with parameter in `[0, 1]` lies on the closed segment.
theorem point2_param_line_on_segment[T: OrderedField](a: Point2[T], b: Point2[T], t: T) {
    T.0 <= t and t <= T.1 implies a.on_segment(b, a.param_line(b, t))
} by {
    if T.0 <= t and t <= T.1 {
        exists(s: T) {
            T.0 <= s and s <= T.1 and a.param_line(b, t) = a.param_line(b, s)
        }
    }
}

/// The first endpoint lies on the closed segment.
theorem point2_on_segment_start[T: OrderedField](a: Point2[T], b: Point2[T]) {
    a.on_segment(b, a)
} by {
    point2_param_line_zero(a, b)
    T.0 <= T.0
    T.0 <= T.1
    point2_param_line_on_segment(a, b, T.0)
}

/// The second endpoint lies on the closed segment.
theorem point2_on_segment_end[T: OrderedField](a: Point2[T], b: Point2[T]) {
    a.on_segment(b, b)
} by {
    point2_param_line_one(a, b)
    T.0 <= T.1
    T.1 <= T.1
    point2_param_line_on_segment(a, b, T.1)
}

/// Segment membership implies line membership.
theorem point2_on_segment_imp_on_line[T: OrderedField](a: Point2[T], b: Point2[T], p: Point2[T]) {
    a.on_segment(b, p) implies a.on_line(b, p)
} by {
    if a.on_segment(b, p) {
        let t: T satisfy {
            T.0 <= t and t <= T.1 and p = a.param_line(b, t)
        }
        point2_param_line_collinear(a, b, t)
        a.collinear(b, p)
    }
}

/// Betweenness is closed-segment membership by definition.
theorem point2_between_iff_on_segment[T: OrderedField](a: Point2[T], p: Point2[T], b: Point2[T]) {
    a.between(p, b) = a.on_segment(b, p)
}

/// The first endpoint lies between itself and the second endpoint.
theorem point2_between_start[T: OrderedField](a: Point2[T], b: Point2[T]) {
    a.between(a, b)
} by {
    point2_on_segment_start(a, b)
}

/// The second endpoint lies between the first endpoint and itself.
theorem point2_between_end[T: OrderedField](a: Point2[T], b: Point2[T]) {
    a.between(b, b)
} by {
    point2_on_segment_end(a, b)
}

/// Betweenness implies line membership.
theorem point2_between_imp_on_line[T: OrderedField](a: Point2[T], p: Point2[T], b: Point2[T]) {
    a.between(p, b) implies a.on_line(b, p)
} by {
    if a.between(p, b) {
        point2_on_segment_imp_on_line(a, b, p)
    }
}

/// Translating a point on a segment gives a point on the translated segment.
theorem point2_on_segment_translate_forward[T: OrderedField](a: Point2[T], b: Point2[T], p: Point2[T], v: Point2[T]) {
    a.on_segment(b, p) implies a.translate(v).on_segment(b.translate(v), p.translate(v))
} by {
    if a.on_segment(b, p) {
        let t: T satisfy {
            T.0 <= t and t <= T.1 and p = a.param_line(b, t)
        }
        point2_param_line_translate(a, b, t, v)
        exists(s: T) {
            T.0 <= s and s <= T.1 and
            p.translate(v) = a.translate(v).param_line(b.translate(v), s)
        }
    }
}

/// Translating all points preserves segment membership.
theorem point2_on_segment_translate[T: OrderedField](a: Point2[T], b: Point2[T], p: Point2[T], v: Point2[T]) {
    a.translate(v).on_segment(b.translate(v), p.translate(v)) = a.on_segment(b, p)
} by {
    if a.on_segment(b, p) {
        point2_on_segment_translate_forward(a, b, p, v)
    }
    if a.translate(v).on_segment(b.translate(v), p.translate(v)) {
        point2_on_segment_translate_forward(a.translate(v), b.translate(v), p.translate(v), v.neg)
        point2_translate_neg_right(a, v)
        point2_translate_neg_right(b, v)
        point2_translate_neg_right(p, v)
        a.on_segment(b, p)
    }
}

/// Segment membership is symmetric in the two endpoints.
theorem point2_on_segment_swap[T: OrderedField](a: Point2[T], b: Point2[T], p: Point2[T]) {
    a.on_segment(b, p) = b.on_segment(a, p)
} by {
    if a.on_segment(b, p) {
        let t: T satisfy {
            T.0 <= t and t <= T.1 and p = a.param_line(b, t)
        }
        let s = T.1 - t
        point2_unit_sub_bounds(t)
        T.0 <= T.1 - t
        T.1 - t <= T.1
        point2_param_line_reverse(a, b, t)
        b.on_segment(a, p)
    }
    if b.on_segment(a, p) {
        let t: T satisfy {
            T.0 <= t and t <= T.1 and p = b.param_line(a, t)
        }
        let s = T.1 - t
        point2_unit_sub_bounds(t)
        T.0 <= T.1 - t
        T.1 - t <= T.1
        point2_param_line_reverse(b, a, t)
        a.on_segment(b, p)
    }
}

/// Betweenness is symmetric in its endpoints.
theorem point2_between_swap[T: OrderedField](a: Point2[T], p: Point2[T], b: Point2[T]) {
    a.between(p, b) = b.between(p, a)
} by {
    point2_on_segment_swap(a, b, p)
}

/// Translating all points preserves betweenness.
theorem point2_between_translate[T: OrderedField](a: Point2[T], p: Point2[T], b: Point2[T], v: Point2[T]) {
    a.translate(v).between(p.translate(v), b.translate(v)) = a.between(p, b)
} by {
    point2_on_segment_translate(a, b, p, v)
}
