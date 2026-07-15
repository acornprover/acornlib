from comm_ring import CommRing
from ordered_field import OrderedField
from geometry.point2 import Point2, point2_zero, point2_add_neg_right
from geometry.point2_algebra import point2_norm_sq_add_expansion,
    point2_dist_sq_comm, point2_dist_sq_zero_right,
    point2_norm_sq_neg, point2_norm_sq_smul, point2_sub_reverse_neg
from geometry.point2_affine import point2_add_sub_left_cancel,
    point2_dist_sq_translate, point2_param_line_reverse,
    point2_param_line_translate, point2_sub_param_line_start
from geometry.point2_circle import point2_on_circle_translate_forward
from geometry.point2_segment import point2_between_iff_on_segment,
    point2_on_segment_translate_forward

/// The scalar normalization used in the parameterized chord power identity.
theorem point2_chord_scalar_power_identity[T: CommRing](d2: T, ad: T, t: T) {
    d2 + ad + ad = T.0 implies
        T.0 - (t * t * d2 + t * ad + t * ad) = t * (T.1 - t) * d2
} by {
    if d2 + ad + ad = T.0 {
        (-d2 + d2) + (ad + ad) = T.0 + (ad + ad)
        t * ad + t * ad = -(t * d2)
        t * t * d2 + t * ad + t * ad = t * t * d2 - t * d2
        t * d2 - t * t * d2 = t * (T.1 - t) * d2
        T.0 - (t * t * d2 + t * ad + t * ad) = t * (T.1 - t) * d2
    }
}

/// The scalar normalization used for squared parameter lengths on a chord.
theorem point2_chord_scalar_product_square[T: CommRing](d2: T, t: T) {
    (t * t * d2) * ((T.1 - t) * (T.1 - t) * d2) =
        (t * (T.1 - t) * d2) * (t * (T.1 - t) * d2)
} by {
    let s = T.1 - t
    (t * t * d2) * (s * s * d2) = (t * t) * (s * s) * (d2 * d2)
    (t * s * d2) * (t * s * d2) = (t * s) * (t * s) * (d2 * d2)
    (t * t) * (s * s) = (t * s) * (t * s)
}

/// The squared distance from a parameter point to the starting endpoint.
theorem point2_param_line_start_dist_sq[T: CommRing](a: Point2[T], b: Point2[T], t: T) {
    a.param_line(b, t).dist_sq(a) = t * t * b.sub(a).norm_sq
} by {
    let d = b.sub(a)
    let p = a.param_line(b, t)
    point2_sub_param_line_start(a, b, t)
    point2_norm_sq_smul(t, d)
}

/// The squared distance from a parameter point to the ending endpoint.
theorem point2_param_line_end_dist_sq[T: CommRing](a: Point2[T], b: Point2[T], t: T) {
    a.param_line(b, t).dist_sq(b) = (T.1 - t) * (T.1 - t) * b.sub(a).norm_sq
} by {
    let d = b.sub(a)
    let p = a.param_line(b, t)
    let s = T.1 - t
    point2_param_line_reverse(a, b, t)
    point2_sub_param_line_start(b, a, s)
    point2_norm_sq_smul(s, a.sub(b))
    point2_sub_reverse_neg(a, b)
    point2_norm_sq_neg(d)
}

/// Translating a point on a circle to the origin gives its squared radius as squared norm.
theorem point2_on_circle_translated_norm_sq[T: OrderedField](center: Point2[T], radius_sq: T, p: Point2[T]) {
    center.on_circle(radius_sq, p) implies p.translate(center.neg).norm_sq = radius_sq
} by {
    if center.on_circle(radius_sq, p) {
        let v = center.neg
        let p0 = p.translate(v)
        let c0 = center.translate(v)
        p.dist_sq(center) = radius_sq
        point2_add_neg_right(center)
        point2_dist_sq_translate(p, center, v)
        point2_dist_sq_zero_right(p0)
        p.translate(center.neg).norm_sq = radius_sq
    }
}

/// The power of a parameter point on an equal-norm chord.
theorem point2_chord_param_power[T: CommRing](a: Point2[T], b: Point2[T], t: T) {
    a.norm_sq = b.norm_sq implies
        a.norm_sq - a.param_line(b, t).norm_sq =
        t * (T.1 - t) * b.sub(a).norm_sq
} by {
    if a.norm_sq = b.norm_sq {
        let d = b.sub(a)
        let p = a.param_line(b, t)
        p = a.add(d.smul(t))
        point2_add_sub_left_cancel(a, b)
        b = a.add(d)
        point2_norm_sq_add_expansion(a, d)
        b.norm_sq = a.norm_sq + d.norm_sq + a.dot(d) + a.dot(d)
        a.norm_sq = a.norm_sq + d.norm_sq + a.dot(d) + a.dot(d)
        let e = d.norm_sq + a.dot(d) + a.dot(d)
        a.norm_sq + e = a.norm_sq + d.norm_sq + a.dot(d) + a.dot(d)
        a.norm_sq = a.norm_sq + e
        a.norm_sq + e = a.norm_sq
        (a.norm_sq + e) - a.norm_sq = a.norm_sq - a.norm_sq
        (a.norm_sq + e) - a.norm_sq = e
        a.norm_sq - a.norm_sq = T.0
        e = T.0
        d.norm_sq + a.dot(d) + a.dot(d) = T.0
        point2_norm_sq_add_expansion(a, d.smul(t))
        p.norm_sq = a.norm_sq + d.smul(t).norm_sq + a.dot(d.smul(t)) + a.dot(d.smul(t))
        point2_norm_sq_smul(t, d)
        d.smul(t).norm_sq = t * t * d.norm_sq
        a.dot(d.smul(t)) = t * a.dot(d)
        p.norm_sq = a.norm_sq + t * t * d.norm_sq + t * a.dot(d) + t * a.dot(d)
        let delta = t * t * d.norm_sq + t * a.dot(d) + t * a.dot(d)
        p.norm_sq = a.norm_sq + delta
        a.norm_sq - p.norm_sq = a.norm_sq - (a.norm_sq + delta)
        a.norm_sq - (a.norm_sq + delta) = T.0 - delta
        point2_chord_scalar_power_identity(d.norm_sq, a.dot(d), t)
        a.norm_sq - p.norm_sq = t * (T.1 - t) * d.norm_sq
        a.norm_sq - a.param_line(b, t).norm_sq = t * (T.1 - t) * b.sub(a).norm_sq
    }
}

/// The squared-distance product for a parameter point on a chord is the square of its power.
theorem point2_chord_param_power_square[T: CommRing](a: Point2[T], b: Point2[T], t: T) {
    a.norm_sq = b.norm_sq implies
        a.param_line(b, t).dist_sq(a) * a.param_line(b, t).dist_sq(b) =
        (a.norm_sq - a.param_line(b, t).norm_sq) *
        (a.norm_sq - a.param_line(b, t).norm_sq)
} by {
    if a.norm_sq = b.norm_sq {
        let d = b.sub(a)
        let p = a.param_line(b, t)
        point2_param_line_start_dist_sq(a, b, t)
        point2_param_line_end_dist_sq(a, b, t)
        point2_chord_param_power(a, b, t)
        point2_chord_scalar_product_square(d.norm_sq, t)
        a.param_line(b, t).dist_sq(a) * a.param_line(b, t).dist_sq(b) =
            (a.norm_sq - a.param_line(b, t).norm_sq) *
            (a.norm_sq - a.param_line(b, t).norm_sq)
    }
}

/// Intersecting parameterized chords of one circle have equal products of squared segment lengths.
theorem point2_intersecting_chords_dist_sq_product_eq[T: OrderedField](
    center: Point2[T], radius_sq: T,
    a: Point2[T], b: Point2[T], c: Point2[T], d: Point2[T], p: Point2[T],
    u: T, v: T
) {
    center.on_circle(radius_sq, a) and
    center.on_circle(radius_sq, b) and
    center.on_circle(radius_sq, c) and
    center.on_circle(radius_sq, d) and
    p = a.param_line(b, u) and
    p = c.param_line(d, v)
    implies
    a.dist_sq(p) * b.dist_sq(p) = c.dist_sq(p) * d.dist_sq(p)
} by {
    if center.on_circle(radius_sq, a) and
        center.on_circle(radius_sq, b) and
        center.on_circle(radius_sq, c) and
        center.on_circle(radius_sq, d) and
        p = a.param_line(b, u) and
        p = c.param_line(d, v) {
        let w = center.neg
        let a0 = a.translate(w)
        let b0 = b.translate(w)
        let c0 = c.translate(w)
        let d0 = d.translate(w)
        let p0 = p.translate(w)

        point2_param_line_translate(a, b, u, w)
        point2_param_line_translate(c, d, v, w)

        point2_on_circle_translated_norm_sq(center, radius_sq, a)
        point2_on_circle_translated_norm_sq(center, radius_sq, b)
        point2_on_circle_translated_norm_sq(center, radius_sq, c)
        point2_on_circle_translated_norm_sq(center, radius_sq, d)
        a0.norm_sq = radius_sq
        b0.norm_sq = radius_sq
        c0.norm_sq = radius_sq
        d0.norm_sq = radius_sq
        a0.norm_sq = b0.norm_sq
        c0.norm_sq = d0.norm_sq

        point2_chord_param_power_square(a0, b0, u)
        point2_chord_param_power_square(c0, d0, v)
        p0.dist_sq(a0) * p0.dist_sq(b0) = p0.dist_sq(c0) * p0.dist_sq(d0)

        point2_dist_sq_comm(p0, a0)
        point2_dist_sq_comm(p0, b0)
        point2_dist_sq_comm(p0, c0)
        point2_dist_sq_comm(p0, d0)
        point2_dist_sq_translate(a, p, w)
        point2_dist_sq_translate(b, p, w)
        point2_dist_sq_translate(c, p, w)
        point2_dist_sq_translate(d, p, w)
        a.dist_sq(p) * b.dist_sq(p) = c.dist_sq(p) * d.dist_sq(p)
    }
}

/// Intersecting chords expressed by segment membership have equal products of squared segment lengths.
theorem point2_intersecting_chords_on_segments_dist_sq_product_eq[T: OrderedField](
    center: Point2[T], radius_sq: T,
    a: Point2[T], b: Point2[T], c: Point2[T], d: Point2[T], p: Point2[T]
) {
    center.on_circle(radius_sq, a) and
    center.on_circle(radius_sq, b) and
    center.on_circle(radius_sq, c) and
    center.on_circle(radius_sq, d) and
    a.on_segment(b, p) and
    c.on_segment(d, p)
    implies
    a.dist_sq(p) * b.dist_sq(p) = c.dist_sq(p) * d.dist_sq(p)
} by {
    let u: T satisfy {
        T.0 <= u and u <= T.1 and p = a.param_line(b, u)
    }
    let v: T satisfy {
        T.0 <= v and v <= T.1 and p = c.param_line(d, v)
    }
    point2_intersecting_chords_dist_sq_product_eq(center, radius_sq, a, b, c, d, p, u, v)
}

/// Intersecting chords expressed by betweenness have equal products of squared segment lengths.
theorem point2_intersecting_chords_between_dist_sq_product_eq[T: OrderedField](
    center: Point2[T], radius_sq: T,
    a: Point2[T], b: Point2[T], c: Point2[T], d: Point2[T], p: Point2[T]
) {
    center.on_circle(radius_sq, a) and
    center.on_circle(radius_sq, b) and
    center.on_circle(radius_sq, c) and
    center.on_circle(radius_sq, d) and
    a.between(p, b) and
    c.between(p, d)
    implies
    a.dist_sq(p) * b.dist_sq(p) = c.dist_sq(p) * d.dist_sq(p)
} by {
    point2_between_iff_on_segment(a, p, b)
    point2_between_iff_on_segment(c, p, d)
    point2_intersecting_chords_on_segments_dist_sq_product_eq(center, radius_sq, a, b, c, d, p)
}

/// Translating a segment-membership chord configuration preserves the chord product identity.
theorem point2_intersecting_chords_on_segments_translate_dist_sq_product_eq[T: OrderedField](
    center: Point2[T], radius_sq: T,
    a: Point2[T], b: Point2[T], c: Point2[T], d: Point2[T], p: Point2[T], v: Point2[T]
) {
    center.on_circle(radius_sq, a) and
    center.on_circle(radius_sq, b) and
    center.on_circle(radius_sq, c) and
    center.on_circle(radius_sq, d) and
    a.on_segment(b, p) and
    c.on_segment(d, p)
    implies
    a.translate(v).dist_sq(p.translate(v)) * b.translate(v).dist_sq(p.translate(v)) =
    c.translate(v).dist_sq(p.translate(v)) * d.translate(v).dist_sq(p.translate(v))
} by {
    point2_on_circle_translate_forward(center, radius_sq, a, v)
    point2_on_circle_translate_forward(center, radius_sq, b, v)
    point2_on_circle_translate_forward(center, radius_sq, c, v)
    point2_on_circle_translate_forward(center, radius_sq, d, v)
    center.translate(v).on_circle(radius_sq, b.translate(v))
    center.translate(v).on_circle(radius_sq, c.translate(v))
    center.translate(v).on_circle(radius_sq, d.translate(v))
    point2_on_segment_translate_forward(a, b, p, v)
    point2_on_segment_translate_forward(c, d, p, v)
    point2_intersecting_chords_on_segments_dist_sq_product_eq(
        center.translate(v), radius_sq,
        a.translate(v), b.translate(v), c.translate(v), d.translate(v), p.translate(v))
}
